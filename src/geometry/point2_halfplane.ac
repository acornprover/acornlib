from ordered_field import OrderedField
from algebra.add_ordered_group import neg_le_neg_iff, neg_ge_neg_iff
from data.basic.logic import and_comm, or_comm
from geometry.point2 import Point2
from geometry.point2_affine import point2_orientation_translate
from geometry.point2_segment import point2_on_line_translate
from geometry.point2_side import point2_left_of_line_reverse,
    point2_right_of_line_reverse, point2_left_of_line_translate,
    point2_right_of_line_translate

attributes Point2[T: OrderedField] {
    /// True when a point lies in the open left half-plane of a directed line.
    define in_open_left_halfplane(self, b: Point2[T], p: Point2[T]) -> Bool {
        self.left_of_line(b, p)
    }

    /// True when a point lies in the open right half-plane of a directed line.
    define in_open_right_halfplane(self, b: Point2[T], p: Point2[T]) -> Bool {
        self.right_of_line(b, p)
    }

    /// True when a point lies in the closed left half-plane of a directed line.
    define in_closed_left_halfplane(self, b: Point2[T], p: Point2[T]) -> Bool {
        self.orientation(b, p) >= T.0
    }

    /// True when a point lies in the closed right half-plane of a directed line.
    define in_closed_right_halfplane(self, b: Point2[T], p: Point2[T]) -> Bool {
        self.orientation(b, p) <= T.0
    }

    /// True when a point lies on the boundary line of a directed half-plane.
    define on_halfplane_boundary(self, b: Point2[T], p: Point2[T]) -> Bool {
        self.on_line(b, p)
    }

    /// True when two points lie in a common closed half-plane of a directed line.
    define same_closed_halfplane(self, b: Point2[T], p: Point2[T], q: Point2[T]) -> Bool {
        (self.in_closed_left_halfplane(b, p) and self.in_closed_left_halfplane(b, q)) or
        (self.in_closed_right_halfplane(b, p) and self.in_closed_right_halfplane(b, q))
    }

    /// True when two points lie in opposite open half-planes of a directed line.
    define opposite_open_halfplanes(self, b: Point2[T], p: Point2[T], q: Point2[T]) -> Bool {
        (self.in_open_left_halfplane(b, p) and self.in_open_right_halfplane(b, q)) or
        (self.in_open_right_halfplane(b, p) and self.in_open_left_halfplane(b, q))
    }
}

/// Open left half-plane membership is strict left-of-line membership.
theorem point2_open_left_halfplane_eq_left_of_line[T: OrderedField](a: Point2[T], b: Point2[T], p: Point2[T]) {
    a.in_open_left_halfplane(b, p) = a.left_of_line(b, p)
}

/// Open right half-plane membership is strict right-of-line membership.
theorem point2_open_right_halfplane_eq_right_of_line[T: OrderedField](a: Point2[T], b: Point2[T], p: Point2[T]) {
    a.in_open_right_halfplane(b, p) = a.right_of_line(b, p)
}

/// Open left half-plane membership is the left-turn predicate.
theorem point2_open_left_halfplane_eq_left_turn[T: OrderedField](a: Point2[T], b: Point2[T], p: Point2[T]) {
    a.in_open_left_halfplane(b, p) = a.left_turn(b, p)
}

/// Open right half-plane membership is the right-turn predicate.
theorem point2_open_right_halfplane_eq_right_turn[T: OrderedField](a: Point2[T], b: Point2[T], p: Point2[T]) {
    a.in_open_right_halfplane(b, p) = a.right_turn(b, p)
}

/// Boundary membership is line membership.
theorem point2_on_halfplane_boundary_eq_on_line[T: OrderedField](a: Point2[T], b: Point2[T], p: Point2[T]) {
    a.on_halfplane_boundary(b, p) = a.on_line(b, p)
}

/// Boundary membership is collinearity with the directed line endpoints.
theorem point2_on_halfplane_boundary_eq_collinear[T: OrderedField](a: Point2[T], b: Point2[T], p: Point2[T]) {
    a.on_halfplane_boundary(b, p) = a.collinear(b, p)
}

/// A boundary point lies in the closed left half-plane.
theorem point2_boundary_in_closed_left_halfplane[T: OrderedField](a: Point2[T], b: Point2[T], p: Point2[T]) {
    a.on_halfplane_boundary(b, p) implies a.in_closed_left_halfplane(b, p)
} by {
    if a.on_halfplane_boundary(b, p) {
        a.collinear(b, p)
        a.orientation(b, p) >= T.0
    }
}

/// A boundary point lies in the closed right half-plane.
theorem point2_boundary_in_closed_right_halfplane[T: OrderedField](a: Point2[T], b: Point2[T], p: Point2[T]) {
    a.on_halfplane_boundary(b, p) implies a.in_closed_right_halfplane(b, p)
} by {
    if a.on_halfplane_boundary(b, p) {
        a.collinear(b, p)
        a.orientation(b, p) <= T.0
    }
}

/// A point in the open left half-plane lies in the closed left half-plane.
theorem point2_open_left_in_closed_left_halfplane[T: OrderedField](a: Point2[T], b: Point2[T], p: Point2[T]) {
    a.in_open_left_halfplane(b, p) implies a.in_closed_left_halfplane(b, p)
} by {
    if a.in_open_left_halfplane(b, p) {
        a.left_turn(b, p)
        a.orientation(b, p) >= T.0
    }
}

/// A point in the open right half-plane lies in the closed right half-plane.
theorem point2_open_right_in_closed_right_halfplane[T: OrderedField](a: Point2[T], b: Point2[T], p: Point2[T]) {
    a.in_open_right_halfplane(b, p) implies a.in_closed_right_halfplane(b, p)
} by {
    if a.in_open_right_halfplane(b, p) {
        a.right_turn(b, p)
        a.orientation(b, p) <= T.0
    }
}

/// Reversing the directed line turns open left half-plane membership into open right half-plane membership.
theorem point2_open_left_halfplane_reverse[T: OrderedField](a: Point2[T], b: Point2[T], p: Point2[T]) {
    b.in_open_left_halfplane(a, p) = a.in_open_right_halfplane(b, p)
} by {
    point2_left_of_line_reverse(a, b, p)
}

/// Reversing the directed line turns open right half-plane membership into open left half-plane membership.
theorem point2_open_right_halfplane_reverse[T: OrderedField](a: Point2[T], b: Point2[T], p: Point2[T]) {
    b.in_open_right_halfplane(a, p) = a.in_open_left_halfplane(b, p)
} by {
    point2_right_of_line_reverse(a, b, p)
}

/// Reversing the directed line turns closed left half-plane membership into closed right half-plane membership.
theorem point2_closed_left_halfplane_reverse[T: OrderedField](a: Point2[T], b: Point2[T], p: Point2[T]) {
    b.in_closed_left_halfplane(a, p) = a.in_closed_right_halfplane(b, p)
} by {
    b.in_closed_left_halfplane(a, p) = (b.orientation(a, p) >= T.0)
    a.in_closed_right_halfplane(b, p) = (a.orientation(b, p) <= T.0)
    b.orientation(a, p) = -a.orientation(b, p)
    neg_ge_neg_iff(T.0, a.orientation(b, p))
    -T.0 = T.0
}

/// Reversing the directed line turns closed right half-plane membership into closed left half-plane membership.
theorem point2_closed_right_halfplane_reverse[T: OrderedField](a: Point2[T], b: Point2[T], p: Point2[T]) {
    b.in_closed_right_halfplane(a, p) = a.in_closed_left_halfplane(b, p)
} by {
    neg_le_neg_iff(T.0, a.orientation(b, p))
}

/// Common closed half-plane membership is symmetric in the two witness points.
theorem point2_same_closed_halfplane_swap_points[T: OrderedField](a: Point2[T], b: Point2[T], p: Point2[T], q: Point2[T]) {
    a.same_closed_halfplane(b, q, p) = a.same_closed_halfplane(b, p, q)
} by {
    let lp = a.in_closed_left_halfplane(b, p)
    let rp = a.in_closed_right_halfplane(b, p)
    let lq = a.in_closed_left_halfplane(b, q)
    let rq = a.in_closed_right_halfplane(b, q)
    and_comm(lq, lp)
    and_comm(rq, rp)
}

/// Common closed half-plane membership is invariant under reversing the directed line.
theorem point2_same_closed_halfplane_reverse_line[T: OrderedField](a: Point2[T], b: Point2[T], p: Point2[T], q: Point2[T]) {
    b.same_closed_halfplane(a, p, q) = a.same_closed_halfplane(b, p, q)
} by {
    point2_closed_left_halfplane_reverse(a, b, p)
    point2_closed_left_halfplane_reverse(a, b, q)
    point2_closed_right_halfplane_reverse(a, b, p)
    point2_closed_right_halfplane_reverse(a, b, q)
    let lp = a.in_closed_left_halfplane(b, p)
    let rp = a.in_closed_right_halfplane(b, p)
    let lq = a.in_closed_left_halfplane(b, q)
    let rq = a.in_closed_right_halfplane(b, q)
    or_comm(rp and rq, lp and lq)
}

/// Opposite open half-plane membership is symmetric in the two witness points.
theorem point2_opposite_open_halfplanes_swap_points[T: OrderedField](a: Point2[T], b: Point2[T], p: Point2[T], q: Point2[T]) {
    a.opposite_open_halfplanes(b, q, p) = a.opposite_open_halfplanes(b, p, q)
} by {
    let lp = a.in_open_left_halfplane(b, p)
    let rp = a.in_open_right_halfplane(b, p)
    let lq = a.in_open_left_halfplane(b, q)
    let rq = a.in_open_right_halfplane(b, q)
    and_comm(lq, rp)
    and_comm(rq, lp)
    or_comm(rp and lq, lp and rq)
}

/// Opposite open half-plane membership is invariant under reversing the directed line.
theorem point2_opposite_open_halfplanes_reverse_line[T: OrderedField](a: Point2[T], b: Point2[T], p: Point2[T], q: Point2[T]) {
    b.opposite_open_halfplanes(a, p, q) = a.opposite_open_halfplanes(b, p, q)
} by {
    point2_open_left_halfplane_reverse(a, b, p)
    point2_open_left_halfplane_reverse(a, b, q)
    point2_open_right_halfplane_reverse(a, b, p)
    point2_open_right_halfplane_reverse(a, b, q)
    let lp = a.in_open_left_halfplane(b, p)
    let rp = a.in_open_right_halfplane(b, p)
    let lq = a.in_open_left_halfplane(b, q)
    let rq = a.in_open_right_halfplane(b, q)
    or_comm(rp and lq, lp and rq)
}

/// Translating all points preserves open left half-plane membership.
theorem point2_open_left_halfplane_translate[T: OrderedField](a: Point2[T], b: Point2[T], p: Point2[T], v: Point2[T]) {
    a.translate(v).in_open_left_halfplane(b.translate(v), p.translate(v)) =
    a.in_open_left_halfplane(b, p)
} by {
    point2_left_of_line_translate(a, b, p, v)
}

/// Translating all points preserves open right half-plane membership.
theorem point2_open_right_halfplane_translate[T: OrderedField](a: Point2[T], b: Point2[T], p: Point2[T], v: Point2[T]) {
    a.translate(v).in_open_right_halfplane(b.translate(v), p.translate(v)) =
    a.in_open_right_halfplane(b, p)
} by {
    point2_right_of_line_translate(a, b, p, v)
}

/// Translating all points preserves closed left half-plane membership.
theorem point2_closed_left_halfplane_translate[T: OrderedField](a: Point2[T], b: Point2[T], p: Point2[T], v: Point2[T]) {
    a.translate(v).in_closed_left_halfplane(b.translate(v), p.translate(v)) =
    a.in_closed_left_halfplane(b, p)
} by {
    point2_orientation_translate(a, b, p, v)
}

/// Translating all points preserves closed right half-plane membership.
theorem point2_closed_right_halfplane_translate[T: OrderedField](a: Point2[T], b: Point2[T], p: Point2[T], v: Point2[T]) {
    a.translate(v).in_closed_right_halfplane(b.translate(v), p.translate(v)) =
    a.in_closed_right_halfplane(b, p)
} by {
    point2_orientation_translate(a, b, p, v)
}

/// Translating all points preserves boundary membership.
theorem point2_halfplane_boundary_translate[T: OrderedField](a: Point2[T], b: Point2[T], p: Point2[T], v: Point2[T]) {
    a.translate(v).on_halfplane_boundary(b.translate(v), p.translate(v)) =
    a.on_halfplane_boundary(b, p)
} by {
    point2_on_line_translate(a, b, p, v)
}

/// Translating all points preserves common closed half-plane membership.
theorem point2_same_closed_halfplane_translate[T: OrderedField](a: Point2[T], b: Point2[T], p: Point2[T], q: Point2[T], v: Point2[T]) {
    a.translate(v).same_closed_halfplane(b.translate(v), p.translate(v), q.translate(v)) =
    a.same_closed_halfplane(b, p, q)
} by {
    point2_closed_left_halfplane_translate(a, b, p, v)
    point2_closed_left_halfplane_translate(a, b, q, v)
    point2_closed_right_halfplane_translate(a, b, p, v)
    point2_closed_right_halfplane_translate(a, b, q, v)
}

/// Translating all points preserves opposite open half-plane membership.
theorem point2_opposite_open_halfplanes_translate[T: OrderedField](a: Point2[T], b: Point2[T], p: Point2[T], q: Point2[T], v: Point2[T]) {
    a.translate(v).opposite_open_halfplanes(b.translate(v), p.translate(v), q.translate(v)) =
    a.opposite_open_halfplanes(b, p, q)
} by {
    point2_open_left_halfplane_translate(a, b, p, v)
    point2_open_left_halfplane_translate(a, b, q, v)
    point2_open_right_halfplane_translate(a, b, p, v)
    point2_open_right_halfplane_translate(a, b, q, v)
}
