/// Pick's theorem (Freek Top 100 #92).
///
/// For a simple lattice polygon — a polygon whose vertices are lattice points of
/// the integer grid — the area equals
///
///     Area = I + B / 2 - 1
///
/// where `I` is the number of lattice points in the interior of the polygon and
/// `B` is the number of lattice points on its boundary.
///
/// Status: the general theorem is not yet provable here (it needs a notion of
/// "simple polygon", the finiteness of the lattice points inside an arbitrary
/// polygon, and the triangulation/ehrhart machinery; none of these exist in the
/// library yet).  This file contributes the clean definitions — lattice points,
/// lattice polygons, the (doubled) shoelace area, boundary and closed-region
/// containment — together with fully verified instances of Pick's formula for
/// three concrete lattice polygons: the unit square, the 1-by-2 rectangle, and
/// the 3-4-5 right triangle.  The general statement is recorded in a comment at
/// the end of the file.

from nat import Nat, from_nat, from_nat_add, from_nat_mul, from_nat_one, from_nat_zero
from int import Int
from rat import Rat
from real import Real, mul_inverse, mul_div_cancel, mul_one_over, mul_div_left,
    pos_gt_zero, gt_zero_imp_pos, mul_pos_pos, lt_add_pos, lt_trans,
    from_nat_is_from_rat
from list import List
from order import lt_imp_ne
from finite_set import FiniteSet, fs_insert, fs_union, finite_set_empty_contains_eq,
    finite_set_insert_cardinality_is_suc_of_not_contains, finite_set_ext,
    finite_set_ext_contains, finite_set_union_contains_eq,
    finite_set_disjoint_union_cardinality_is
from data.finite.finite_set_card import fs_card, fs_card_eq_of_cardinality_is,
    fs_card_cardinality_is, fs_card_empty
from data.finite.finite_set_membership import fs_insert_contains_eq
from data.finite.finite_set_filter import finite_set_filter,
    finite_set_filter_contains_eq
from data.basic.logic import exists_intro, or_false
from data.basic.set import set_ext
from geometry.point2 import Point2, point2_ext, point2_new_x, point2_new_y
from geometry.point2_triangle import point2_triangle_area2_eq_orientation
from geometry.point2_polygon import point2_polygon_area2, point2_polygon_area2_quad,
    point2_polygon_area2_triple, point2_polygon_ccw
from geometry.point2_polygon_reverse import point2_list_last_or
from geometry.point2_polygon_edges import point2_polyline_contains,
    point2_polyline_contains_first
from geometry.point2_halfplane import point2_closed_left_halfplane_translate
from geometry.point2_incidence_extra import point2_orientation_coordinate_formula

numerals Real
numerals Nat

// ---------------------------------------------------------------------------
// Small real constants used by the coordinate computations.
// ---------------------------------------------------------------------------

/// The real number two.
let real_two: Real = Real.1 + Real.1

/// The real number three.
let real_three: Real = real_two + Real.1

/// The real number four.
let real_four: Real = real_three + Real.1

/// The real number six.
let real_six: Real = real_four + real_two

/// The real number eight.
let real_eight: Real = real_six + real_two

/// The real number twelve.
let real_twelve: Real = real_four * real_three

// ---------------------------------------------------------------------------
// Order, division, and embedding arithmetic for these constants.
// ---------------------------------------------------------------------------

/// The real number two is positive.
theorem real_two_pos {
    Real.0 < real_two
} by {
    Real.1.is_positive
    pos_gt_zero(Real.1)
    Real.0 < Real.1
    Real.1.is_positive
    lt_add_pos(Real.1, Real.1)
    Real.1 < Real.1 + Real.1
    Real.1 < real_two
    Real.0 < real_two
}

/// The real number two is nonzero.
theorem real_two_ne_zero {
    real_two != Real.0
} by {
    Real.0 < real_two
    lt_imp_ne(Real.0, real_two)
}

/// Zero is not one.
theorem real_ne_zero_one {
    Real.0 != Real.1
} by {
    Real.1.is_positive
    pos_gt_zero(Real.1)
    Real.0 < Real.1
    lt_imp_ne(Real.0, Real.1)
}

/// One is not zero.
theorem real_ne_one_zero {
    Real.1 != Real.0
} by {
    real_ne_zero_one
    Real.1 != Real.0
}

/// One is less than two.
theorem real_lt_1_2 {
    Real.1 < real_two
} by {
    Real.1.is_positive
    lt_add_pos(Real.1, Real.1)
    Real.1 < Real.1 + Real.1
    Real.1 < real_two
}

/// Two is less than three.
theorem real_lt_2_3 {
    real_two < real_three
} by {
    Real.1.is_positive
    lt_add_pos(real_two, Real.1)
    real_two < real_two + Real.1
    real_two < real_three
}

/// Three is less than four.
theorem real_lt_3_4 {
    real_three < real_four
} by {
    Real.1.is_positive
    lt_add_pos(real_three, Real.1)
    real_three < real_three + Real.1
    real_three < real_four
}

/// Zero is less than one.
theorem real_lt_0_1 {
    Real.0 < Real.1
} by {
    Real.1.is_positive
    pos_gt_zero(Real.1)
    Real.0 < Real.1
}

/// Zero is less than three.
theorem real_lt_0_3 {
    Real.0 < real_three
} by {
    real_two_pos
    Real.0 < real_two
    real_lt_2_3
    real_two < real_three
    lt_trans(Real.0, real_two, real_three)
    Real.0 < real_three
}

/// Two is less than four.
theorem real_lt_2_4 {
    real_two < real_four
} by {
    real_lt_2_3
    real_two < real_three
    real_lt_3_4
    real_three < real_four
    lt_trans(real_two, real_three, real_four)
    real_two < real_four
}

/// Zero is less than four.
theorem real_lt_0_4 {
    Real.0 < real_four
} by {
    real_two_pos
    Real.0 < real_two
    real_lt_2_4
    real_two < real_four
    lt_trans(Real.0, real_two, real_four)
    Real.0 < real_four
}

/// One is less than three.
theorem real_lt_1_3 {
    Real.1 < real_three
} by {
    real_lt_1_2
    Real.1 < real_two
    real_lt_2_3
    real_two < real_three
    lt_trans(Real.1, real_two, real_three)
    Real.1 < real_three
}

/// One is less than four.
theorem real_lt_1_4 {
    Real.1 < real_four
} by {
    real_lt_1_2
    Real.1 < real_two
    real_lt_2_4
    real_two < real_four
    lt_trans(Real.1, real_two, real_four)
    Real.1 < real_four
}

/// Zero is not two.
theorem real_ne_0_2 {
    Real.0 != real_two
} by {
    real_two_pos
    lt_imp_ne(Real.0, real_two)
}

/// Two is not zero.
theorem real_ne_2_0 {
    real_two != Real.0
} by {
    real_ne_0_2
    real_two != Real.0
}

/// One is not two.
theorem real_ne_1_2 {
    Real.1 != real_two
} by {
    real_lt_1_2
    lt_imp_ne(Real.1, real_two)
}

/// Two is not one.
theorem real_ne_2_1 {
    real_two != Real.1
} by {
    real_ne_1_2
    real_two != Real.1
}

/// Zero is not three.
theorem real_ne_0_3 {
    Real.0 != real_three
} by {
    real_lt_0_3
    lt_imp_ne(Real.0, real_three)
}

/// Three is not zero.
theorem real_ne_3_0 {
    real_three != Real.0
} by {
    real_ne_0_3
    real_three != Real.0
}

/// One is not three.
theorem real_ne_1_3 {
    Real.1 != real_three
} by {
    real_lt_1_3
    lt_imp_ne(Real.1, real_three)
}

/// Three is not one.
theorem real_ne_3_1 {
    real_three != Real.1
} by {
    real_ne_1_3
    real_three != Real.1
}

/// Two is not three.
theorem real_ne_2_3 {
    real_two != real_three
} by {
    real_lt_2_3
    lt_imp_ne(real_two, real_three)
}

/// Three is not two.
theorem real_ne_3_2 {
    real_three != real_two
} by {
    real_ne_2_3
    real_three != real_two
}

/// Zero is not four.
theorem real_ne_0_4 {
    Real.0 != real_four
} by {
    real_lt_0_4
    lt_imp_ne(Real.0, real_four)
}

/// Four is not zero.
theorem real_ne_4_0 {
    real_four != Real.0
} by {
    real_ne_0_4
    real_four != Real.0
}

/// One is not four.
theorem real_ne_1_4 {
    Real.1 != real_four
} by {
    real_lt_1_4
    lt_imp_ne(Real.1, real_four)
}

/// Four is not one.
theorem real_ne_4_1 {
    real_four != Real.1
} by {
    real_ne_1_4
    real_four != Real.1
}

/// Two is not four.
theorem real_ne_2_4 {
    real_two != real_four
} by {
    real_lt_2_4
    lt_imp_ne(real_two, real_four)
}

/// Four is not two.
theorem real_ne_4_2 {
    real_four != real_two
} by {
    real_ne_2_4
    real_four != real_two
}

/// Three is not four.
theorem real_ne_3_4 {
    real_three != real_four
} by {
    real_lt_3_4
    lt_imp_ne(real_three, real_four)
}

/// Four is not three.
theorem real_ne_4_3 {
    real_four != real_three
} by {
    real_ne_3_4
    real_four != real_three
}

/// Division distributes over addition on the left.
theorem div_add_distrib_local(a: Real, b: Real, c: Real) {
    c != Real.0 implies (a + b) / c = a / c + b / c
} by {
    if c != Real.0 {
        (a + b) / c = (a + b) * c.inverse
        (a + b) * c.inverse = a * c.inverse + b * c.inverse
        a / c = a * c.inverse
        b / c = b * c.inverse
        (a + b) / c = a / c + b / c
    }
}

/// Two divided by two is one.
theorem real_two_div_two {
    real_two / real_two = Real.1
} by {
    real_two / real_two = real_two * real_two.inverse
    mul_inverse(real_two)
    real_two * real_two.inverse = Real.1
    real_two / real_two = Real.1
}

/// Four divided by two is two.
theorem real_four_div_two {
    real_four / real_two = real_two
} by {
    real_four = real_two + real_two
    real_four / real_two = (real_two + real_two) / real_two
    div_add_distrib_local(real_two, real_two, real_two)
    real_two != Real.0
    (real_two + real_two) / real_two = real_two / real_two + real_two / real_two
    real_two / real_two = Real.1
    real_two / real_two + real_two / real_two = Real.1 + Real.1
    Real.1 + Real.1 = real_two
    real_four / real_two = real_two
}

/// Six divided by two is three.
theorem real_six_div_two {
    real_six / real_two = real_three
} by {
    real_six = real_four + real_two
    real_six / real_two = (real_four + real_two) / real_two
    div_add_distrib_local(real_four, real_two, real_two)
    real_two != Real.0
    (real_four + real_two) / real_two = real_four / real_two + real_two / real_two
    real_four_div_two
    real_four / real_two = real_two
    real_two_div_two
    real_two / real_two = Real.1
    real_four / real_two + real_two / real_two = real_two + Real.1
    real_two + Real.1 = real_three
    real_six / real_two = real_three
}

/// Eight divided by two is four.
theorem real_eight_div_two {
    real_eight / real_two = real_four
} by {
    real_eight = real_six + real_two
    real_eight / real_two = (real_six + real_two) / real_two
    div_add_distrib_local(real_six, real_two, real_two)
    real_two != Real.0
    (real_six + real_two) / real_two = real_six / real_two + real_two / real_two
    real_six_div_two
    real_six / real_two = real_three
    real_two_div_two
    real_two / real_two = Real.1
    real_six / real_two + real_two / real_two = real_three + Real.1
    real_three + Real.1 = real_four
    real_eight / real_two = real_four
}

/// Two embeds as the real number two.
theorem from_nat_two {
    from_nat[Real](Nat.2) = real_two
} by {
    from_nat_add[Real](Nat.1, Nat.1)
    from_nat[Real](Nat.1 + Nat.1) = from_nat[Real](Nat.1) + from_nat[Real](Nat.1)
    Nat.1 + Nat.1 = Nat.2
    from_nat[Real](Nat.2) = from_nat[Real](Nat.1) + from_nat[Real](Nat.1)
    from_nat_one[Real]
    from_nat[Real](Nat.1) = Real.1
    from_nat[Real](Nat.1) + from_nat[Real](Nat.1) = Real.1 + Real.1
    Real.1 + Real.1 = real_two
    from_nat[Real](Nat.2) = real_two
}

/// Three embeds as the real number three.
theorem from_nat_three {
    from_nat[Real](Nat.3) = real_three
} by {
    from_nat_add[Real](Nat.2, Nat.1)
    from_nat[Real](Nat.2 + Nat.1) = from_nat[Real](Nat.2) + from_nat[Real](Nat.1)
    Nat.2 + Nat.1 = Nat.3
    from_nat[Real](Nat.3) = from_nat[Real](Nat.2) + from_nat[Real](Nat.1)
    from_nat_two
    from_nat[Real](Nat.2) = real_two
    from_nat[Real](Nat.2) + from_nat[Real](Nat.1) = real_two + Real.1
    real_two + Real.1 = real_three
    from_nat[Real](Nat.3) = real_three
}

/// Four embeds as the real number four.
theorem from_nat_four {
    from_nat[Real](Nat.4) = real_four
} by {
    from_nat_add[Real](Nat.2, Nat.2)
    from_nat[Real](Nat.2 + Nat.2) = from_nat[Real](Nat.2) + from_nat[Real](Nat.2)
    Nat.2 + Nat.2 = Nat.4
    from_nat[Real](Nat.4) = from_nat[Real](Nat.2) + from_nat[Real](Nat.2)
    from_nat_two
    from_nat[Real](Nat.2) = real_two
    from_nat[Real](Nat.2) + from_nat[Real](Nat.2) = real_two + real_two
    real_two + real_two = real_four
    from_nat[Real](Nat.4) = real_four
}

/// Six embeds as the real number six.
theorem from_nat_six {
    from_nat[Real](Nat.6) = real_six
} by {
    from_nat_add[Real](Nat.4, Nat.2)
    from_nat[Real](Nat.4 + Nat.2) = from_nat[Real](Nat.4) + from_nat[Real](Nat.2)
    Nat.4 + Nat.2 = Nat.6
    from_nat[Real](Nat.6) = from_nat[Real](Nat.4) + from_nat[Real](Nat.2)
    from_nat_four
    from_nat_two
    from_nat[Real](Nat.4) + from_nat[Real](Nat.2) = real_four + real_two
    real_four + real_two = real_six
    from_nat[Real](Nat.6) = real_six
}

/// Eight embeds as the real number eight.
theorem from_nat_eight {
    from_nat[Real](Nat.8) = real_eight
} by {
    from_nat_add[Real](Nat.6, Nat.2)
    from_nat[Real](Nat.6 + Nat.2) = from_nat[Real](Nat.6) + from_nat[Real](Nat.2)
    Nat.6 + Nat.2 = Nat.8
    from_nat[Real](Nat.8) = from_nat[Real](Nat.6) + from_nat[Real](Nat.2)
    from_nat_six
    from_nat_two
    from_nat[Real](Nat.6) + from_nat[Real](Nat.2) = real_six + real_two
    real_six + real_two = real_eight
    from_nat[Real](Nat.8) = real_eight
}

/// The embedding of `Int.from_nat(n)` into the reals is `from_nat[Real](n)`.
theorem real_from_int_from_nat(n: Nat) {
    Real.from_int(Int.from_nat(n)) = from_nat[Real](n)
} by {
    Real.from_int(Int.from_nat(n)) = Real.from_rat(Rat.from_int(Int.from_nat(n)))
    Real.from_rat(Rat.from_int(Int.from_nat(n))) = Real.from_rat(Rat.from_nat(n))
    Real.from_rat(Rat.from_nat(n)) = from_nat[Real](n)
    Real.from_int(Int.from_nat(n)) = from_nat[Real](n)
}

// ---------------------------------------------------------------------------
// Lattice points and lattice polygons.
// ---------------------------------------------------------------------------

/// True when the coordinates of `p` are the integer embeddings `m` and `n`.
define is_lattice_coords(p: Point2[Real], m: Int, n: Int) -> Bool {
    p.x = Real.from_int(m) and p.y = Real.from_int(n)
}

/// True when a point of the coordinate plane has integer coordinates.
define is_lattice_point(p: Point2[Real]) -> Bool {
    exists(m: Int, n: Int) {
        is_lattice_coords(p, m, n)
    }
}

/// A point whose coordinates are integer embeddings is a lattice point.
theorem is_lattice_point_of_coords(p: Point2[Real], m: Int, n: Int) {
    p.x = Real.from_int(m) and p.y = Real.from_int(n) implies is_lattice_point(p)
} by {
    if p.x = Real.from_int(m) and p.y = Real.from_int(n) {
        is_lattice_point(p) = exists(x: Int, y: Int) {
            is_lattice_coords(p, x, y)
        }
        is_lattice_coords(p, m, n) = (p.x = Real.from_int(m) and p.y = Real.from_int(n))
        is_lattice_coords(p, m, n)
        exists_intro(function(y: Int) { is_lattice_coords(p, m, y) }, n)
        exists(y: Int) { is_lattice_coords(p, m, y) }
        exists_intro(function(x: Int) { exists(y: Int) { is_lattice_coords(p, x, y) } }, m)
        exists(x: Int, y: Int) { is_lattice_coords(p, x, y) }
        is_lattice_point(p)
    }
}

/// True when every vertex of a polygonal point list is a lattice point.
define is_lattice_polygon(points: List[Point2[Real]]) -> Bool {
    match points {
        List.nil {
            true
        }
        List.cons(head, tail) {
            is_lattice_point(head) and is_lattice_polygon(tail)
        }
    }
}

/// The empty point list is trivially a lattice polygon.
theorem is_lattice_polygon_nil {
    is_lattice_polygon(List.nil[Point2[Real]])
}

/// A lattice polygon cons step keeps the head's lattice membership.
theorem is_lattice_polygon_intro(head: Point2[Real], tail: List[Point2[Real]]) {
    (is_lattice_point(head) and is_lattice_polygon(tail)) implies
    is_lattice_polygon(List.cons(head, tail))
} by {
    if is_lattice_point(head) and is_lattice_polygon(tail) {
        is_lattice_polygon(List.cons(head, tail))
    }
}

/// True for a counter-clockwise lattice polygon.
///
/// The library has no general notion of "simple polygon", so positivity of the
/// signed doubled shoelace area (`point2_polygon_ccw`) stands in for simplicity:
/// a polygon with positive doubled area winds once counter-clockwise around a
/// bounded region, which is exactly the class for which the shoelace formula
/// gives the ordinary area.
define is_simple_lattice_polygon(points: List[Point2[Real]]) -> Bool {
    is_lattice_polygon(points) and point2_polygon_ccw(points)
}

// ---------------------------------------------------------------------------
// Area.
// ---------------------------------------------------------------------------

/// The signed doubled area of a polygonal point list (the shoelace sum).
define polygon_area2(points: List[Point2[Real]]) -> Real {
    point2_polygon_area2(points)
}

/// The signed area of a polygonal point list, one half of the shoelace sum.
///
/// For a counter-clockwise polygon this is the ordinary area; the library's
/// `point2_polygon_area2` computes the doubled signed area, so the area is that
/// value divided by two.
define polygon_area(points: List[Point2[Real]]) -> Real {
    point2_polygon_area2(points) / real_two
}

// ---------------------------------------------------------------------------
// Boundary and closed-region containment for polygons.
// ---------------------------------------------------------------------------

/// True when `p` lies on one of the segments of the chain `points`, including
/// the closing segment from the last vertex back to `first`.
define point_on_polygon_edges(first: Point2[Real], points: List[Point2[Real]], p: Point2[Real]) -> Bool {
    match points {
        List.nil {
            false
        }
        List.cons(head, tail) {
            match tail {
                List.nil {
                    head.on_segment(first, p)
                }
                List.cons(second, rest) {
                    if head.on_segment(second, p) {
                        true
                    } else {
                        point_on_polygon_edges(first, tail, p)
                    }
                }
            }
        }
    }
}

/// True when `p` lies on one of the segments of a closed polygon, including the
/// closing edge from the last vertex back to the first.
define point_on_polygon_boundary(points: List[Point2[Real]], p: Point2[Real]) -> Bool {
    match points {
        List.nil {
            false
        }
        List.cons(first, tail) {
            match tail {
                List.nil {
                    false
                }
                List.cons(second, rest) {
                    point_on_polygon_edges(first, points, p)
                }
            }
        }
    }
}

/// The edge-by-edge closed-left-half-plane test for a counter-clockwise polygon.
///
/// `first` is the first vertex of the polygon; the last edge checked is the
/// closing edge from the last vertex back to `first`.
define point2_polygon_edges_left(first: Point2[Real], points: List[Point2[Real]], p: Point2[Real]) -> Bool {
    match points {
        List.nil {
            false
        }
        List.cons(head, tail) {
            match tail {
                List.nil {
                    head.in_closed_left_halfplane(first, p)
                }
                List.cons(second, rest) {
                    head.in_closed_left_halfplane(second, p) and
                    point2_polygon_edges_left(first, tail, p)
                }
            }
        }
    }
}

/// True when `p` lies in the closed region of a counter-clockwise polygon.
///
/// For a convex counter-clockwise polygon the closed region is exactly the
/// intersection of the closed left half-planes of the directed edges, including
/// the closing edge.  This is the containment predicate used by the statement of
/// Pick's theorem: the interior is the region minus the boundary.
define point_in_polygon_region(points: List[Point2[Real]], p: Point2[Real]) -> Bool {
    match points {
        List.nil {
            true
        }
        List.cons(first, tail) {
            match tail {
                List.nil {
                    true
                }
                List.cons(second, rest) {
                    point2_polygon_edges_left(first, points, p)
                }
            }
        }
    }
}

/// The number of lattice points in a finite point set.
///
/// `I` and `B` in Pick's theorem are instances of this count: for a polygon,
/// `interior_lattice_count` is `lattice_point_count` of the (finite) set of
/// lattice points strictly inside it, and `boundary_lattice_count` is
/// `lattice_point_count` of the lattice points on its boundary.  The library
/// cannot yet construct those two finite sets for an arbitrary polygon, so the
/// instances below are given by explicit finite sets.
define lattice_point_count(s: FiniteSet[Point2[Real]]) -> Nat {
    fs_card(finite_set_filter(s, is_lattice_point))
}

/// A finite point set consisting entirely of lattice points is counted exactly.
theorem lattice_point_count_of_lattice_set(s: FiniteSet[Point2[Real]]) {
    (forall(p: Point2[Real]) {
        s.contains(p) implies is_lattice_point(p)
    }) implies lattice_point_count(s) = fs_card(s)
} by {
    if forall(p: Point2[Real]) {
        s.contains(p) implies is_lattice_point(p)
    } {
        let filtered = finite_set_filter(s, is_lattice_point)
        forall(x: Point2[Real]) {
            finite_set_filter_contains_eq(s, is_lattice_point, x)
            filtered.contains(x) = (s.contains(x) and is_lattice_point(x))
            if filtered.contains(x) {
                s.contains(x)
            }
            if s.contains(x) {
                is_lattice_point(x)
                filtered.contains(x)
            }
            filtered.contains(x) = s.contains(x)
            filtered.underlying_set.contains(x) = s.underlying_set.contains(x)
        }
        set_ext(filtered.underlying_set, s.underlying_set)
        filtered.underlying_set = s.underlying_set
        finite_set_ext(filtered, s)
        filtered = s
        lattice_point_count(s) = fs_card(filtered)
        lattice_point_count(s) = fs_card(s)
    }
}

/// Points with different x-coordinates are distinct.
theorem point2_ne_of_x_ne(p: Point2[Real], q: Point2[Real]) {
    p.x != q.x implies p != q
} by {
    if p.x != q.x {
        if p = q {
            p.x = q.x
            false
        }
        p != q
    }
}

/// Points with different y-coordinates are distinct.
theorem point2_ne_of_y_ne(p: Point2[Real], q: Point2[Real]) {
    p.y != q.y implies p != q
} by {
    if p.y != q.y {
        if p = q {
            p.y = q.y
            false
        }
        p != q
    }
}

// ---------------------------------------------------------------------------
// Case 1: the unit square [0,1] x [0,1].
//
//   q01 ---- q11
//   |         |
//   |         |
//   q00 ---- q10
//
// Area 1, I = 0 (no strictly interior lattice points), B = 4 (the corners).
// Pick: 0 + 4/2 - 1 = 1.
// ---------------------------------------------------------------------------

/// The lower-left corner of the unit square.
let q00: Point2[Real] = Point2.new(Real.0, Real.0)

/// The lower-right corner of the unit square.
let q10: Point2[Real] = Point2.new(Real.1, Real.0)

/// The upper-right corner of the unit square.
let q11: Point2[Real] = Point2.new(Real.1, Real.1)

/// The upper-left corner of the unit square.
let q01: Point2[Real] = Point2.new(Real.0, Real.1)

/// The unit square as a counter-clockwise vertex list.
let unit_square: List[Point2[Real]] =
    List.cons(q00, List.cons(q10, List.cons(q11, List.cons(q01, List.nil[Point2[Real]]))))

/// The boundary lattice points of the unit square: its four corners.
let unit_square_boundary: FiniteSet[Point2[Real]] =
    fs_insert(fs_insert(fs_insert(fs_insert(FiniteSet.empty[Point2[Real]], q01), q11), q10), q00)

/// The interior lattice points of the unit square: none.
let unit_square_interior: FiniteSet[Point2[Real]] = FiniteSet.empty[Point2[Real]]

/// The four corners of the unit square are lattice points.
theorem is_lattice_point_q00 {
    is_lattice_point(q00)
} by {
    q00.x = Real.0
    Real.from_int(Int.0) = Real.0
    q00.x = Real.from_int(Int.0)
    q00.y = Real.0
    q00.y = Real.from_int(Int.0)
    q00.x = Real.from_int(Int.0) and q00.y = Real.from_int(Int.0)
    is_lattice_point_of_coords(q00, Int.0, Int.0)
    is_lattice_point(q00)
}

/// The four corners of the unit square are lattice points.
theorem is_lattice_point_q10 {
    is_lattice_point(q10)
} by {
    q10.x = Real.1
    Real.from_int(Int.1) = Real.1
    q10.x = Real.from_int(Int.1)
    q10.y = Real.0
    q10.y = Real.from_int(Int.0)
    q10.x = Real.from_int(Int.1) and q10.y = Real.from_int(Int.0)
    is_lattice_point_of_coords(q10, Int.1, Int.0)
    is_lattice_point(q10)
}

/// The four corners of the unit square are lattice points.
theorem is_lattice_point_q11 {
    is_lattice_point(q11)
} by {
    q11.x = Real.1
    Real.from_int(Int.1) = Real.1
    q11.x = Real.from_int(Int.1)
    q11.y = Real.1
    q11.y = Real.from_int(Int.1)
    q11.x = Real.from_int(Int.1) and q11.y = Real.from_int(Int.1)
    is_lattice_point_of_coords(q11, Int.1, Int.1)
    is_lattice_point(q11)
}

/// The four corners of the unit square are lattice points.
theorem is_lattice_point_q01 {
    is_lattice_point(q01)
} by {
    q01.x = Real.0
    Real.from_int(Int.0) = Real.0
    q01.x = Real.from_int(Int.0)
    q01.y = Real.1
    q01.y = Real.from_int(Int.1)
    q01.x = Real.from_int(Int.0) and q01.y = Real.from_int(Int.1)
    is_lattice_point_of_coords(q01, Int.0, Int.1)
    is_lattice_point(q01)
}

/// The unit square is a lattice polygon.
theorem is_lattice_polygon_unit_square {
    is_lattice_polygon(unit_square)
} by {
    is_lattice_polygon_nil
    is_lattice_polygon(List.nil[Point2[Real]])
    is_lattice_polygon_intro(q01, List.nil[Point2[Real]])
    is_lattice_point(q01) and is_lattice_polygon(List.nil[Point2[Real]])
    is_lattice_polygon(List.cons(q01, List.nil[Point2[Real]]))
    is_lattice_polygon_intro(q11, List.cons(q01, List.nil[Point2[Real]]))
    is_lattice_point(q11) and is_lattice_polygon(List.cons(q01, List.nil[Point2[Real]]))
    is_lattice_polygon(List.cons(q11, List.cons(q01, List.nil[Point2[Real]])))
    is_lattice_polygon_intro(q10, List.cons(q11, List.cons(q01, List.nil[Point2[Real]])))
    is_lattice_point(q10) and is_lattice_polygon(List.cons(q11, List.cons(q01, List.nil[Point2[Real]])))
    is_lattice_polygon(List.cons(q10, List.cons(q11, List.cons(q01, List.nil[Point2[Real]]))))
    is_lattice_polygon_intro(q00, List.cons(q10, List.cons(q11, List.cons(q01, List.nil[Point2[Real]]))))
    is_lattice_point(q00) and is_lattice_polygon(List.cons(q10, List.cons(q11, List.cons(q01, List.nil[Point2[Real]]))))
    is_lattice_polygon(unit_square)
}

/// The doubled shoelace area of the unit square is two.
theorem unit_square_area2_is_two {
    point2_polygon_area2(unit_square) = real_two
} by {
    point2_polygon_area2_quad(q00, q10, q11, q01)
    point2_polygon_area2(unit_square) =
        q00.triangle_area2(q10, q11) + q00.triangle_area2(q11, q01)
    point2_triangle_area2_eq_orientation(q00, q10, q11)
    point2_orientation_coordinate_formula(q00, q10, q11)
    point2_new_x(Real.0, Real.0)
    point2_new_y(Real.0, Real.0)
    point2_new_x(Real.1, Real.0)
    point2_new_y(Real.1, Real.0)
    point2_new_x(Real.1, Real.1)
    point2_new_y(Real.1, Real.1)
    q00.x = Real.0
    q00.y = Real.0
    q10.x = Real.1
    q10.y = Real.0
    q11.x = Real.1
    q11.y = Real.1
    q00.orientation(q10, q11) = (Real.1 - Real.0) * (Real.1 - Real.0) - (Real.0 - Real.0) * (Real.1 - Real.0)
    (Real.1 - Real.0) * (Real.1 - Real.0) - (Real.0 - Real.0) * (Real.1 - Real.0) = Real.1
    q00.triangle_area2(q10, q11) = Real.1
    point2_triangle_area2_eq_orientation(q00, q11, q01)
    point2_orientation_coordinate_formula(q00, q11, q01)
    point2_new_x(Real.0, Real.1)
    point2_new_y(Real.0, Real.1)
    q01.x = Real.0
    q01.y = Real.1
    q00.orientation(q11, q01) = (Real.1 - Real.0) * (Real.1 - Real.0) - (Real.1 - Real.0) * (Real.0 - Real.0)
    (Real.1 - Real.0) * (Real.1 - Real.0) - (Real.1 - Real.0) * (Real.0 - Real.0) = Real.1
    q00.triangle_area2(q11, q01) = Real.1
    q00.triangle_area2(q10, q11) + q00.triangle_area2(q11, q01) = Real.1 + Real.1
    Real.1 + Real.1 = real_two
    point2_polygon_area2(unit_square) = real_two
}

/// The unit square winds counter-clockwise (its doubled area is positive).
theorem unit_square_ccw {
    point2_polygon_ccw(unit_square)
} by {
    point2_polygon_area2(unit_square) = real_two
    real_two_pos
    Real.0 < real_two
    point2_polygon_area2(unit_square) > Real.0
    point2_polygon_ccw(unit_square) = (point2_polygon_area2(unit_square) > Real.0)
    point2_polygon_ccw(unit_square)
}

/// The unit square is a simple lattice polygon.
theorem is_simple_lattice_polygon_unit_square {
    is_simple_lattice_polygon(unit_square)
} by {
    is_lattice_polygon_unit_square
    unit_square_ccw
    is_simple_lattice_polygon(unit_square) =
        (is_lattice_polygon(unit_square) and point2_polygon_ccw(unit_square))
    is_lattice_polygon(unit_square) and point2_polygon_ccw(unit_square)
    is_simple_lattice_polygon(unit_square)
}

/// The area of the unit square is one.
theorem unit_square_area_is_one {
    polygon_area(unit_square) = Real.1
} by {
    unit_square_area2_is_two
    point2_polygon_area2(unit_square) = real_two
    polygon_area(unit_square) = point2_polygon_area2(unit_square) / real_two
    polygon_area(unit_square) = real_two / real_two
    real_two_div_two
    real_two / real_two = Real.1
    polygon_area(unit_square) = Real.1
}

/// The four corners of the unit square are distinct.
theorem unit_square_corners_distinct {
    q00 != q10 and q10 != q11 and q11 != q01 and q01 != q00 and
    q00 != q11 and q10 != q01
} by {
    point2_ne_of_x_ne(q00, q10)
    q00.x = Real.0
    q10.x = Real.1
    real_ne_zero_one
    Real.0 != Real.1
    q00.x != q10.x
    q00 != q10
    point2_ne_of_y_ne(q10, q11)
    q10.y = Real.0
    q11.y = Real.1
    real_ne_zero_one
    Real.0 != Real.1
    q10.y != q11.y
    q10 != q11
    point2_ne_of_x_ne(q11, q01)
    q11.x = Real.1
    q01.x = Real.0
    real_ne_one_zero
    Real.1 != Real.0
    q11.x != q01.x
    q11 != q01
    point2_ne_of_y_ne(q01, q00)
    q01.y = Real.1
    q00.y = Real.0
    real_ne_one_zero
    Real.1 != Real.0
    q01.y != q00.y
    q01 != q00
    point2_ne_of_x_ne(q00, q11)
    q00.x = Real.0
    q11.x = Real.1
    real_ne_zero_one
    Real.0 != Real.1
    q00.x != q11.x
    q00 != q11
    point2_ne_of_y_ne(q10, q01)
    q10.y = Real.0
    q01.y = Real.1
    real_ne_zero_one
    Real.0 != Real.1
    q10.y != q01.y
    q10 != q01
    q00 != q10 and q10 != q11 and q11 != q01 and q01 != q00 and
        q00 != q11 and q10 != q01
}

/// The unit square has exactly four boundary lattice points.
theorem unit_square_boundary_card {
    fs_card(unit_square_boundary) = Nat.4
} by {
    fs_card_empty[Point2[Real]]
    fs_card(FiniteSet.empty[Point2[Real]]) = Nat.0
    fs_card_cardinality_is(FiniteSet.empty[Point2[Real]])
    FiniteSet.empty[Point2[Real]].cardinality_is(fs_card(FiniteSet.empty[Point2[Real]]))
    FiniteSet.empty[Point2[Real]].cardinality_is(Nat.0)
    finite_set_empty_contains_eq(q01)
    not FiniteSet.empty[Point2[Real]].contains(q01)
    finite_set_insert_cardinality_is_suc_of_not_contains(FiniteSet.empty[Point2[Real]], q01, Nat.0)
    fs_insert(FiniteSet.empty[Point2[Real]], q01).cardinality_is(Nat.1)
    unit_square_corners_distinct
    q11 != q01
    fs_insert_contains_eq(FiniteSet.empty[Point2[Real]], q01, q11)
    fs_insert(FiniteSet.empty[Point2[Real]], q01).contains(q11) =
        (q11 = q01 or FiniteSet.empty[Point2[Real]].contains(q11))
    finite_set_empty_contains_eq(q11)
    FiniteSet.empty[Point2[Real]].contains(q11) = false
    not fs_insert(FiniteSet.empty[Point2[Real]], q01).contains(q11)
    finite_set_insert_cardinality_is_suc_of_not_contains(
        fs_insert(FiniteSet.empty[Point2[Real]], q01), q11, Nat.1)
    fs_insert(fs_insert(FiniteSet.empty[Point2[Real]], q01), q11).cardinality_is(Nat.2)
    q10 != q01
    q10 != q11
    fs_insert_contains_eq(fs_insert(FiniteSet.empty[Point2[Real]], q01), q11, q10)
    fs_insert(fs_insert(FiniteSet.empty[Point2[Real]], q01), q11).contains(q10) =
        (q10 = q11 or fs_insert(FiniteSet.empty[Point2[Real]], q01).contains(q10))
    fs_insert_contains_eq(FiniteSet.empty[Point2[Real]], q01, q10)
    fs_insert(FiniteSet.empty[Point2[Real]], q01).contains(q10) =
        (q10 = q01 or FiniteSet.empty[Point2[Real]].contains(q10))
    finite_set_empty_contains_eq(q10)
    FiniteSet.empty[Point2[Real]].contains(q10) = false
    not fs_insert(fs_insert(FiniteSet.empty[Point2[Real]], q01), q11).contains(q10)
    finite_set_insert_cardinality_is_suc_of_not_contains(
        fs_insert(fs_insert(FiniteSet.empty[Point2[Real]], q01), q11), q10, Nat.2)
    fs_insert(fs_insert(fs_insert(FiniteSet.empty[Point2[Real]], q01), q11), q10).cardinality_is(Nat.3)
    q00 != q01
    q00 != q11
    q00 != q10
    fs_insert_contains_eq(fs_insert(fs_insert(FiniteSet.empty[Point2[Real]], q01), q11), q10, q00)
    fs_insert(fs_insert(fs_insert(FiniteSet.empty[Point2[Real]], q01), q11), q10).contains(q00) =
        (q00 = q10 or fs_insert(fs_insert(FiniteSet.empty[Point2[Real]], q01), q11).contains(q00))
    fs_insert_contains_eq(fs_insert(FiniteSet.empty[Point2[Real]], q01), q11, q00)
    fs_insert(fs_insert(FiniteSet.empty[Point2[Real]], q01), q11).contains(q00) =
        (q00 = q11 or fs_insert(FiniteSet.empty[Point2[Real]], q01).contains(q00))
    fs_insert_contains_eq(FiniteSet.empty[Point2[Real]], q01, q00)
    fs_insert(FiniteSet.empty[Point2[Real]], q01).contains(q00) =
        (q00 = q01 or FiniteSet.empty[Point2[Real]].contains(q00))
    finite_set_empty_contains_eq(q00)
    FiniteSet.empty[Point2[Real]].contains(q00) = false
    not fs_insert(fs_insert(fs_insert(FiniteSet.empty[Point2[Real]], q01), q11), q10).contains(q00)
    finite_set_insert_cardinality_is_suc_of_not_contains(
        fs_insert(fs_insert(fs_insert(FiniteSet.empty[Point2[Real]], q01), q11), q10), q00, Nat.3)
    fs_insert(fs_insert(fs_insert(fs_insert(FiniteSet.empty[Point2[Real]], q01), q11), q10), q00).cardinality_is(Nat.3 + Nat.1)
    Nat.3 + Nat.1 = Nat.4
    fs_insert(fs_insert(fs_insert(fs_insert(FiniteSet.empty[Point2[Real]], q01), q11), q10), q00).cardinality_is(Nat.4)
    unit_square_boundary = fs_insert(fs_insert(fs_insert(fs_insert(FiniteSet.empty[Point2[Real]], q01), q11), q10), q00)
    unit_square_boundary.cardinality_is(Nat.4)
    fs_card_eq_of_cardinality_is(unit_square_boundary, Nat.4)
    fs_card(unit_square_boundary) = Nat.4
}

/// The boundary of the unit square contains exactly its four corners.
theorem unit_square_boundary_contains_eq(x: Point2[Real]) {
    unit_square_boundary.contains(x) =
        (x = q00 or x = q10 or x = q11 or x = q01)
} by {
    fs_insert_contains_eq(fs_insert(fs_insert(fs_insert(FiniteSet.empty[Point2[Real]], q01), q11), q10), q00, x)
    unit_square_boundary.contains(x) =
        (x = q00 or fs_insert(fs_insert(fs_insert(FiniteSet.empty[Point2[Real]], q01), q11), q10).contains(x))
    fs_insert_contains_eq(fs_insert(fs_insert(FiniteSet.empty[Point2[Real]], q01), q11), q10, x)
    fs_insert(fs_insert(fs_insert(FiniteSet.empty[Point2[Real]], q01), q11), q10).contains(x) =
        (x = q10 or fs_insert(fs_insert(FiniteSet.empty[Point2[Real]], q01), q11).contains(x))
    fs_insert_contains_eq(fs_insert(FiniteSet.empty[Point2[Real]], q01), q11, x)
    fs_insert(fs_insert(FiniteSet.empty[Point2[Real]], q01), q11).contains(x) =
        (x = q11 or fs_insert(FiniteSet.empty[Point2[Real]], q01).contains(x))
    fs_insert_contains_eq(FiniteSet.empty[Point2[Real]], q01, x)
    fs_insert(FiniteSet.empty[Point2[Real]], q01).contains(x) =
        (x = q01 or FiniteSet.empty[Point2[Real]].contains(x))
    finite_set_empty_contains_eq(x)
    FiniteSet.empty[Point2[Real]].contains(x) = false
    or_false(x = q01)
    (x = q01 or false) = (x = q01)
    unit_square_boundary.contains(x) =
        (x = q00 or (x = q10 or (x = q11 or x = q01)))
}

/// The unit square has no interior lattice points.
theorem unit_square_interior_card {
    fs_card(unit_square_interior) = Nat.0
} by {
    fs_card_empty[Point2[Real]]
    fs_card(unit_square_interior) = Nat.0
}

/// The unit square has four boundary lattice points.
theorem unit_square_boundary_lattice_count {
    lattice_point_count(unit_square_boundary) = Nat.4
} by {
    lattice_point_count_of_lattice_set(unit_square_boundary)
    forall(p: Point2[Real]) {
        if unit_square_boundary.contains(p) {
            unit_square_boundary_contains_eq(p)
            unit_square_boundary.contains(p) =
                (p = q00 or p = q10 or p = q11 or p = q01)
            p = q00 or p = q10 or p = q11 or p = q01
            if p = q00 {
                is_lattice_point_q00
                is_lattice_point(p)
            } else {
                if p = q10 {
                    is_lattice_point_q10
                    is_lattice_point(p)
                } else {
                    if p = q11 {
                        is_lattice_point_q11
                        is_lattice_point(p)
                    } else {
                        p = q01
                        is_lattice_point_q01
                        is_lattice_point(p)
                    }
                }
            }
        }
    }
    lattice_point_count(unit_square_boundary) = fs_card(unit_square_boundary)
    unit_square_boundary_card
    lattice_point_count(unit_square_boundary) = Nat.4
}

/// The empty point set has no lattice points.
theorem lattice_point_count_empty {
    lattice_point_count(FiniteSet.empty[Point2[Real]]) = Nat.0
} by {
    let filtered = finite_set_filter(FiniteSet.empty[Point2[Real]], is_lattice_point)
    lattice_point_count(FiniteSet.empty[Point2[Real]]) = fs_card(filtered)
    forall(x: Point2[Real]) {
        finite_set_filter_contains_eq(FiniteSet.empty[Point2[Real]], is_lattice_point, x)
        filtered.contains(x) =
            (FiniteSet.empty[Point2[Real]].contains(x) and is_lattice_point(x))
        finite_set_empty_contains_eq(x)
        FiniteSet.empty[Point2[Real]].contains(x) = false
        filtered.contains(x) = false
        finite_set_empty_contains_eq(x)
        FiniteSet.empty[Point2[Real]].contains(x) = false
        filtered.contains(x) = FiniteSet.empty[Point2[Real]].contains(x)
        filtered.contains(x) = filtered.underlying_set.contains(x)
        FiniteSet.empty[Point2[Real]].contains(x) = FiniteSet.empty[Point2[Real]].underlying_set.contains(x)
        filtered.underlying_set.contains(x) = FiniteSet.empty[Point2[Real]].underlying_set.contains(x)
    }
    finite_set_ext_contains(filtered, FiniteSet.empty[Point2[Real]])
    filtered = FiniteSet.empty[Point2[Real]]
    fs_card_empty[Point2[Real]]
    fs_card(FiniteSet.empty[Point2[Real]]) = Nat.0
    lattice_point_count(FiniteSet.empty[Point2[Real]]) = Nat.0
}

/// The unit square has no interior lattice points.
theorem unit_square_interior_lattice_count {
    lattice_point_count(unit_square_interior) = Nat.0
} by {
    unit_square_interior = FiniteSet.empty[Point2[Real]]
    lattice_point_count_empty
    lattice_point_count(unit_square_interior) = Nat.0
}

/// Pick's theorem for the unit square:
/// `Area = 1 = I + B/2 - 1 = 0 + 4/2 - 1`.
theorem pick_unit_square {
    is_simple_lattice_polygon(unit_square) implies
    polygon_area(unit_square) =
        from_nat[Real](lattice_point_count(unit_square_interior)) +
        from_nat[Real](lattice_point_count(unit_square_boundary)) / real_two - Real.1
} by {
    if is_simple_lattice_polygon(unit_square) {
        unit_square_area_is_one
        polygon_area(unit_square) = Real.1
        unit_square_interior_lattice_count
        lattice_point_count(unit_square_interior) = Nat.0
        unit_square_boundary_lattice_count
        lattice_point_count(unit_square_boundary) = Nat.4
        from_nat_zero[Real]
        from_nat[Real](Nat.0) = Real.0
        from_nat[Real](lattice_point_count(unit_square_interior)) = Real.0
        from_nat_four
        from_nat[Real](Nat.4) = real_four
        from_nat[Real](lattice_point_count(unit_square_boundary)) = real_four
        from_nat[Real](lattice_point_count(unit_square_boundary)) / real_two = real_four / real_two
        real_four_div_two
        real_four / real_two = real_two
        from_nat[Real](lattice_point_count(unit_square_boundary)) / real_two = real_two
        from_nat[Real](lattice_point_count(unit_square_interior)) +
            from_nat[Real](lattice_point_count(unit_square_boundary)) / real_two - Real.1 =
            Real.0 + real_two - Real.1
        Real.0 + real_two - Real.1 = Real.1
        polygon_area(unit_square) =
            from_nat[Real](lattice_point_count(unit_square_interior)) +
            from_nat[Real](lattice_point_count(unit_square_boundary)) / real_two - Real.1
    }
}

/// Pick's theorem for the unit square in the library's doubled-area form:
/// `2 * Area = 2I + B - 2 = 0 + 4 - 2`.
theorem pick_unit_square_doubled {
    is_simple_lattice_polygon(unit_square) implies
    point2_polygon_area2(unit_square) =
        from_nat[Real](lattice_point_count(unit_square_interior) +
            lattice_point_count(unit_square_interior) +
            lattice_point_count(unit_square_boundary)) - real_two
} by {
    if is_simple_lattice_polygon(unit_square) {
        unit_square_area2_is_two
        point2_polygon_area2(unit_square) = real_two
        unit_square_interior_lattice_count
        lattice_point_count(unit_square_interior) = Nat.0
        unit_square_boundary_lattice_count
        lattice_point_count(unit_square_boundary) = Nat.4
        lattice_point_count(unit_square_interior) +
            lattice_point_count(unit_square_interior) +
            lattice_point_count(unit_square_boundary) = Nat.4
        from_nat_four
        from_nat[Real](Nat.4) = real_four
        from_nat[Real](lattice_point_count(unit_square_interior) +
            lattice_point_count(unit_square_interior) +
            lattice_point_count(unit_square_boundary)) = real_four
        from_nat[Real](lattice_point_count(unit_square_interior) +
            lattice_point_count(unit_square_interior) +
            lattice_point_count(unit_square_boundary)) - real_two = real_four - real_two
        real_four = real_two + real_two
        real_four - real_two = (real_two + real_two) - real_two
        (real_two + real_two) - real_two = real_two
        real_four - real_two = real_two
        point2_polygon_area2(unit_square) =
            from_nat[Real](lattice_point_count(unit_square_interior) +
                lattice_point_count(unit_square_interior) +
                lattice_point_count(unit_square_boundary)) - real_two
    }
}

// ---------------------------------------------------------------------------
// Case 2: the 1-by-2 rectangle [0,2] x [0,1].
//
//   r01 ---- r11 ---- r21
//   |         |         |
//   r00 ---- r10 ---- r20
//
// Area 2, I = 0 (no strictly interior lattice points), B = 6.
// Pick: 0 + 6/2 - 1 = 2.
// ---------------------------------------------------------------------------

/// The lower-left corner of the rectangle.
let r00: Point2[Real] = Point2.new(Real.0, Real.0)

/// The bottom-middle lattice point of the rectangle.
let r10: Point2[Real] = Point2.new(Real.1, Real.0)

/// The lower-right corner of the rectangle.
let r20: Point2[Real] = Point2.new(real_two, Real.0)

/// The upper-right corner of the rectangle.
let r21: Point2[Real] = Point2.new(real_two, Real.1)

/// The top-middle lattice point of the rectangle.
let r11: Point2[Real] = Point2.new(Real.1, Real.1)

/// The upper-left corner of the rectangle.
let r01: Point2[Real] = Point2.new(Real.0, Real.1)

/// The 1-by-2 rectangle as a counter-clockwise vertex list.
let unit_rect: List[Point2[Real]] =
    List.cons(r00, List.cons(r20, List.cons(r21, List.cons(r01, List.nil[Point2[Real]]))))

/// The boundary lattice points of the rectangle: its six grid points.
let unit_rect_boundary: FiniteSet[Point2[Real]] =
    fs_insert(fs_insert(fs_insert(fs_insert(fs_insert(fs_insert(
        FiniteSet.empty[Point2[Real]], r01), r11), r21), r20), r10), r00)

/// The interior lattice points of the rectangle: none.
let unit_rect_interior: FiniteSet[Point2[Real]] = FiniteSet.empty[Point2[Real]]

/// The six boundary points of the rectangle are lattice points.
theorem is_lattice_point_r00 {
    is_lattice_point(r00)
} by {
    r00.x = Real.0
    Real.from_int(Int.0) = Real.0
    r00.x = Real.from_int(Int.0)
    r00.y = Real.0
    r00.y = Real.from_int(Int.0)
    r00.x = Real.from_int(Int.0) and r00.y = Real.from_int(Int.0)
    is_lattice_point_of_coords(r00, Int.0, Int.0)
    is_lattice_point(r00)
}

/// The six boundary points of the rectangle are lattice points.
theorem is_lattice_point_r10 {
    is_lattice_point(r10)
} by {
    r10.x = Real.1
    Real.from_int(Int.1) = Real.1
    r10.x = Real.from_int(Int.1)
    r10.y = Real.0
    r10.y = Real.from_int(Int.0)
    r10.x = Real.from_int(Int.1) and r10.y = Real.from_int(Int.0)
    is_lattice_point_of_coords(r10, Int.1, Int.0)
    is_lattice_point(r10)
}

/// The six boundary points of the rectangle are lattice points.
theorem is_lattice_point_r20 {
    is_lattice_point(r20)
} by {
    r20.x = real_two
    real_from_int_from_nat(Nat.2)
    from_nat_two
    Real.from_int(Int.from_nat(Nat.2)) = real_two
    r20.x = Real.from_int(Int.from_nat(Nat.2))
    r20.y = Real.0
    Real.from_int(Int.0) = Real.0
    r20.y = Real.from_int(Int.0)
    r20.x = Real.from_int(Int.from_nat(Nat.2)) and r20.y = Real.from_int(Int.0)
    is_lattice_point_of_coords(r20, Int.from_nat(Nat.2), Int.0)
    is_lattice_point(r20)
}

/// The six boundary points of the rectangle are lattice points.
theorem is_lattice_point_r21 {
    is_lattice_point(r21)
} by {
    r21.x = real_two
    real_from_int_from_nat(Nat.2)
    from_nat_two
    Real.from_int(Int.from_nat(Nat.2)) = real_two
    r21.x = Real.from_int(Int.from_nat(Nat.2))
    r21.y = Real.1
    Real.from_int(Int.1) = Real.1
    r21.y = Real.from_int(Int.1)
    r21.x = Real.from_int(Int.from_nat(Nat.2)) and r21.y = Real.from_int(Int.1)
    is_lattice_point_of_coords(r21, Int.from_nat(Nat.2), Int.1)
    is_lattice_point(r21)
}

/// The six boundary points of the rectangle are lattice points.
theorem is_lattice_point_r11 {
    is_lattice_point(r11)
} by {
    r11.x = Real.1
    Real.from_int(Int.1) = Real.1
    r11.x = Real.from_int(Int.1)
    r11.y = Real.1
    r11.y = Real.from_int(Int.1)
    r11.x = Real.from_int(Int.1) and r11.y = Real.from_int(Int.1)
    is_lattice_point_of_coords(r11, Int.1, Int.1)
    is_lattice_point(r11)
}

/// The six boundary points of the rectangle are lattice points.
theorem is_lattice_point_r01 {
    is_lattice_point(r01)
} by {
    r01.x = Real.0
    Real.from_int(Int.0) = Real.0
    r01.x = Real.from_int(Int.0)
    r01.y = Real.1
    r01.y = Real.from_int(Int.1)
    r01.x = Real.from_int(Int.0) and r01.y = Real.from_int(Int.1)
    is_lattice_point_of_coords(r01, Int.0, Int.1)
    is_lattice_point(r01)
}

/// The rectangle is a lattice polygon.
theorem is_lattice_polygon_unit_rect {
    is_lattice_polygon(unit_rect)
} by {
    is_lattice_polygon_nil
    is_lattice_polygon(List.nil[Point2[Real]])
    is_lattice_polygon_intro(r01, List.nil[Point2[Real]])
    is_lattice_point(r01) and is_lattice_polygon(List.nil[Point2[Real]])
    is_lattice_polygon(List.cons(r01, List.nil[Point2[Real]]))
    is_lattice_polygon_intro(r21, List.cons(r01, List.nil[Point2[Real]]))
    is_lattice_point(r21) and is_lattice_polygon(List.cons(r01, List.nil[Point2[Real]]))
    is_lattice_polygon(List.cons(r21, List.cons(r01, List.nil[Point2[Real]])))
    is_lattice_polygon_intro(r20, List.cons(r21, List.cons(r01, List.nil[Point2[Real]])))
    is_lattice_point(r20) and is_lattice_polygon(List.cons(r21, List.cons(r01, List.nil[Point2[Real]])))
    is_lattice_polygon(List.cons(r20, List.cons(r21, List.cons(r01, List.nil[Point2[Real]]))))
    is_lattice_polygon_intro(r00, List.cons(r20, List.cons(r21, List.cons(r01, List.nil[Point2[Real]]))))
    is_lattice_point(r00) and is_lattice_polygon(List.cons(r20, List.cons(r21, List.cons(r01, List.nil[Point2[Real]]))))
    is_lattice_polygon(unit_rect)
}

/// The doubled shoelace area of the rectangle is four.
theorem unit_rect_area2_is_four {
    point2_polygon_area2(unit_rect) = real_four
} by {
    point2_polygon_area2_quad(r00, r20, r21, r01)
    point2_polygon_area2(unit_rect) =
        r00.triangle_area2(r20, r21) + r00.triangle_area2(r21, r01)
    point2_triangle_area2_eq_orientation(r00, r20, r21)
    point2_orientation_coordinate_formula(r00, r20, r21)
    point2_new_x(Real.0, Real.0)
    point2_new_y(Real.0, Real.0)
    point2_new_x(real_two, Real.0)
    point2_new_y(real_two, Real.0)
    point2_new_x(real_two, Real.1)
    point2_new_y(real_two, Real.1)
    r00.x = Real.0
    r00.y = Real.0
    r20.x = real_two
    r20.y = Real.0
    r21.x = real_two
    r21.y = Real.1
    r00.orientation(r20, r21) = (real_two - Real.0) * (Real.1 - Real.0) - (Real.0 - Real.0) * (real_two - Real.0)
    (real_two - Real.0) * (Real.1 - Real.0) - (Real.0 - Real.0) * (real_two - Real.0) = real_two
    r00.triangle_area2(r20, r21) = real_two
    point2_triangle_area2_eq_orientation(r00, r21, r01)
    point2_orientation_coordinate_formula(r00, r21, r01)
    point2_new_x(Real.0, Real.1)
    point2_new_y(Real.0, Real.1)
    r01.x = Real.0
    r01.y = Real.1
    r00.orientation(r21, r01) = (real_two - Real.0) * (Real.1 - Real.0) - (Real.1 - Real.0) * (Real.0 - Real.0)
    (real_two - Real.0) * (Real.1 - Real.0) - (Real.1 - Real.0) * (Real.0 - Real.0) = real_two
    r00.triangle_area2(r21, r01) = real_two
    r00.triangle_area2(r20, r21) + r00.triangle_area2(r21, r01) = real_two + real_two
    real_two + real_two = real_four
    point2_polygon_area2(unit_rect) = real_four
}

/// The rectangle winds counter-clockwise.
theorem unit_rect_ccw {
    point2_polygon_ccw(unit_rect)
} by {
    unit_rect_area2_is_four
    point2_polygon_area2(unit_rect) = real_four
    real_lt_0_4
    Real.0 < real_four
    point2_polygon_area2(unit_rect) > Real.0
    point2_polygon_ccw(unit_rect) = (point2_polygon_area2(unit_rect) > Real.0)
    point2_polygon_ccw(unit_rect)
}

/// The rectangle is a simple lattice polygon.
theorem is_simple_lattice_polygon_unit_rect {
    is_simple_lattice_polygon(unit_rect)
} by {
    is_lattice_polygon_unit_rect
    unit_rect_ccw
    is_simple_lattice_polygon(unit_rect) =
        (is_lattice_polygon(unit_rect) and point2_polygon_ccw(unit_rect))
    is_lattice_polygon(unit_rect) and point2_polygon_ccw(unit_rect)
    is_simple_lattice_polygon(unit_rect)
}

/// The area of the rectangle is two.
theorem unit_rect_area_is_two {
    polygon_area(unit_rect) = real_two
} by {
    unit_rect_area2_is_four
    point2_polygon_area2(unit_rect) = real_four
    polygon_area(unit_rect) = point2_polygon_area2(unit_rect) / real_two
    polygon_area(unit_rect) = real_four / real_two
    real_four_div_two
    real_four / real_two = real_two
    polygon_area(unit_rect) = real_two
}

/// The six boundary points of the rectangle are distinct.
theorem unit_rect_points_distinct {
    r00 != r10 and r10 != r20 and r20 != r21 and r21 != r11 and r11 != r01 and
    r00 != r20 and r10 != r21 and r20 != r11 and r21 != r01 and
    r00 != r21 and r10 != r11 and r20 != r01 and
    r00 != r11 and r10 != r01 and
    r00 != r01
} by {
    point2_ne_of_x_ne(r00, r10)
    r00.x = Real.0
    r10.x = Real.1
    real_ne_zero_one
    Real.0 != Real.1
    r00.x != r10.x
    r00 != r10
    point2_ne_of_x_ne(r10, r20)
    r10.x = Real.1
    r20.x = real_two
    real_ne_1_2
    Real.1 != real_two
    r10.x != r20.x
    r10 != r20
    point2_ne_of_y_ne(r20, r21)
    r20.y = Real.0
    r21.y = Real.1
    real_ne_zero_one
    Real.0 != Real.1
    r20.y != r21.y
    r20 != r21
    point2_ne_of_x_ne(r21, r11)
    r21.x = real_two
    r11.x = Real.1
    real_ne_2_1
    real_two != Real.1
    r21.x != r11.x
    r21 != r11
    point2_ne_of_x_ne(r11, r01)
    r11.x = Real.1
    r01.x = Real.0
    real_ne_one_zero
    Real.1 != Real.0
    r11.x != r01.x
    r11 != r01
    point2_ne_of_x_ne(r00, r20)
    r00.x = Real.0
    r20.x = real_two
    real_ne_0_2
    Real.0 != real_two
    r00.x != r20.x
    r00 != r20
    point2_ne_of_x_ne(r10, r21)
    r10.x = Real.1
    r21.x = real_two
    real_ne_1_2
    Real.1 != real_two
    r10.x != r21.x
    r10 != r21
    point2_ne_of_x_ne(r20, r11)
    r20.x = real_two
    r11.x = Real.1
    real_ne_2_1
    real_two != Real.1
    r20.x != r11.x
    r20 != r11
    point2_ne_of_x_ne(r21, r01)
    r21.x = real_two
    r01.x = Real.0
    real_ne_2_0
    real_two != Real.0
    r21.x != r01.x
    r21 != r01
    point2_ne_of_x_ne(r00, r21)
    r00.x = Real.0
    r21.x = real_two
    real_ne_0_2
    Real.0 != real_two
    r00.x != r21.x
    r00 != r21
    point2_ne_of_y_ne(r10, r11)
    r10.y = Real.0
    r11.y = Real.1
    real_ne_zero_one
    Real.0 != Real.1
    r10.y != r11.y
    r10 != r11
    point2_ne_of_x_ne(r20, r01)
    r20.x = real_two
    r01.x = Real.0
    real_ne_2_0
    real_two != Real.0
    r20.x != r01.x
    r20 != r01
    point2_ne_of_x_ne(r00, r11)
    r00.x = Real.0
    r11.x = Real.1
    real_ne_zero_one
    Real.0 != Real.1
    r00.x != r11.x
    r00 != r11
    point2_ne_of_x_ne(r10, r01)
    r10.x = Real.1
    r01.x = Real.0
    real_ne_one_zero
    Real.1 != Real.0
    r10.x != r01.x
    r10 != r01
    point2_ne_of_y_ne(r00, r01)
    r00.y = Real.0
    r01.y = Real.1
    real_ne_zero_one
    Real.0 != Real.1
    r00.y != r01.y
    r00 != r01
    r00 != r10 and r10 != r20 and r20 != r21 and r21 != r11 and r11 != r01 and
        r00 != r20 and r10 != r21 and r20 != r11 and r21 != r01 and
        r00 != r21 and r10 != r11 and r20 != r01 and
        r00 != r11 and r10 != r01 and
        r00 != r01
}

/// The boundary of the rectangle contains exactly its six grid points.
theorem unit_rect_boundary_contains_eq(x: Point2[Real]) {
    unit_rect_boundary.contains(x) =
        (x = r00 or (x = r10 or (x = r20 or (x = r21 or (x = r11 or x = r01)))))
} by {
    fs_insert_contains_eq(fs_insert(fs_insert(fs_insert(fs_insert(fs_insert(
        FiniteSet.empty[Point2[Real]], r01), r11), r21), r20), r10), r00, x)
    unit_rect_boundary.contains(x) =
        (x = r00 or fs_insert(fs_insert(fs_insert(fs_insert(fs_insert(
            FiniteSet.empty[Point2[Real]], r01), r11), r21), r20), r10).contains(x))
    fs_insert_contains_eq(fs_insert(fs_insert(fs_insert(fs_insert(
        FiniteSet.empty[Point2[Real]], r01), r11), r21), r20), r10, x)
    fs_insert(fs_insert(fs_insert(fs_insert(fs_insert(
        FiniteSet.empty[Point2[Real]], r01), r11), r21), r20), r10).contains(x) =
        (x = r10 or fs_insert(fs_insert(fs_insert(fs_insert(
            FiniteSet.empty[Point2[Real]], r01), r11), r21), r20).contains(x))
    fs_insert_contains_eq(fs_insert(fs_insert(fs_insert(
        FiniteSet.empty[Point2[Real]], r01), r11), r21), r20, x)
    fs_insert(fs_insert(fs_insert(fs_insert(
        FiniteSet.empty[Point2[Real]], r01), r11), r21), r20).contains(x) =
        (x = r20 or fs_insert(fs_insert(fs_insert(
            FiniteSet.empty[Point2[Real]], r01), r11), r21).contains(x))
    fs_insert_contains_eq(fs_insert(fs_insert(
        FiniteSet.empty[Point2[Real]], r01), r11), r21, x)
    fs_insert(fs_insert(fs_insert(
        FiniteSet.empty[Point2[Real]], r01), r11), r21).contains(x) =
        (x = r21 or fs_insert(fs_insert(
            FiniteSet.empty[Point2[Real]], r01), r11).contains(x))
    fs_insert_contains_eq(fs_insert(FiniteSet.empty[Point2[Real]], r01), r11, x)
    fs_insert(fs_insert(FiniteSet.empty[Point2[Real]], r01), r11).contains(x) =
        (x = r11 or fs_insert(FiniteSet.empty[Point2[Real]], r01).contains(x))
    fs_insert_contains_eq(FiniteSet.empty[Point2[Real]], r01, x)
    fs_insert(FiniteSet.empty[Point2[Real]], r01).contains(x) =
        (x = r01 or FiniteSet.empty[Point2[Real]].contains(x))
    finite_set_empty_contains_eq(x)
    FiniteSet.empty[Point2[Real]].contains(x) = false
    unit_rect_boundary.contains(x) =
        (x = r00 or (x = r10 or (x = r20 or (x = r21 or (x = r11 or (x = r01 or false))))))
    or_false(x = r01)
    (x = r01 or false) = (x = r01)
    unit_rect_boundary.contains(x) =
        (x = r00 or (x = r10 or (x = r20 or (x = r21 or (x = r11 or x = r01)))))
}

/// The rectangle has exactly six boundary lattice points.
theorem unit_rect_boundary_card {
    fs_card(unit_rect_boundary) = Nat.6
} by {
    fs_card_empty[Point2[Real]]
    fs_card(FiniteSet.empty[Point2[Real]]) = Nat.0
    fs_card_cardinality_is(FiniteSet.empty[Point2[Real]])
    FiniteSet.empty[Point2[Real]].cardinality_is(fs_card(FiniteSet.empty[Point2[Real]]))
    FiniteSet.empty[Point2[Real]].cardinality_is(Nat.0)
    finite_set_empty_contains_eq(r01)
    not FiniteSet.empty[Point2[Real]].contains(r01)
    finite_set_insert_cardinality_is_suc_of_not_contains(FiniteSet.empty[Point2[Real]], r01, Nat.0)
    fs_insert(FiniteSet.empty[Point2[Real]], r01).cardinality_is(Nat.1)
    unit_rect_points_distinct
    r11 != r01
    fs_insert_contains_eq(FiniteSet.empty[Point2[Real]], r01, r11)
    fs_insert(FiniteSet.empty[Point2[Real]], r01).contains(r11) =
        (r11 = r01 or FiniteSet.empty[Point2[Real]].contains(r11))
    finite_set_empty_contains_eq(r11)
    FiniteSet.empty[Point2[Real]].contains(r11) = false
    not fs_insert(FiniteSet.empty[Point2[Real]], r01).contains(r11)
    finite_set_insert_cardinality_is_suc_of_not_contains(
        fs_insert(FiniteSet.empty[Point2[Real]], r01), r11, Nat.1)
    fs_insert(fs_insert(FiniteSet.empty[Point2[Real]], r01), r11).cardinality_is(Nat.2)
    r21 != r01
    r21 != r11
    fs_insert_contains_eq(fs_insert(FiniteSet.empty[Point2[Real]], r01), r11, r21)
    fs_insert(fs_insert(FiniteSet.empty[Point2[Real]], r01), r11).contains(r21) =
        (r21 = r11 or fs_insert(FiniteSet.empty[Point2[Real]], r01).contains(r21))
    fs_insert_contains_eq(FiniteSet.empty[Point2[Real]], r01, r21)
    fs_insert(FiniteSet.empty[Point2[Real]], r01).contains(r21) =
        (r21 = r01 or FiniteSet.empty[Point2[Real]].contains(r21))
    finite_set_empty_contains_eq(r21)
    FiniteSet.empty[Point2[Real]].contains(r21) = false
    not fs_insert(fs_insert(FiniteSet.empty[Point2[Real]], r01), r11).contains(r21)
    finite_set_insert_cardinality_is_suc_of_not_contains(
        fs_insert(fs_insert(FiniteSet.empty[Point2[Real]], r01), r11), r21, Nat.2)
    fs_insert(fs_insert(fs_insert(FiniteSet.empty[Point2[Real]], r01), r11), r21).cardinality_is(Nat.3)
    r20 != r01
    r20 != r11
    r20 != r21
    fs_insert_contains_eq(fs_insert(fs_insert(FiniteSet.empty[Point2[Real]], r01), r11), r21, r20)
    fs_insert(fs_insert(fs_insert(FiniteSet.empty[Point2[Real]], r01), r11), r21).contains(r20) =
        (r20 = r21 or fs_insert(fs_insert(FiniteSet.empty[Point2[Real]], r01), r11).contains(r20))
    fs_insert_contains_eq(fs_insert(FiniteSet.empty[Point2[Real]], r01), r11, r20)
    fs_insert(fs_insert(FiniteSet.empty[Point2[Real]], r01), r11).contains(r20) =
        (r20 = r11 or fs_insert(FiniteSet.empty[Point2[Real]], r01).contains(r20))
    fs_insert_contains_eq(FiniteSet.empty[Point2[Real]], r01, r20)
    fs_insert(FiniteSet.empty[Point2[Real]], r01).contains(r20) =
        (r20 = r01 or FiniteSet.empty[Point2[Real]].contains(r20))
    finite_set_empty_contains_eq(r20)
    FiniteSet.empty[Point2[Real]].contains(r20) = false
    not fs_insert(fs_insert(fs_insert(FiniteSet.empty[Point2[Real]], r01), r11), r21).contains(r20)
    finite_set_insert_cardinality_is_suc_of_not_contains(
        fs_insert(fs_insert(fs_insert(FiniteSet.empty[Point2[Real]], r01), r11), r21), r20, Nat.3)
    fs_insert(fs_insert(fs_insert(fs_insert(FiniteSet.empty[Point2[Real]], r01), r11), r21), r20).cardinality_is(Nat.3 + Nat.1)
    Nat.3 + Nat.1 = Nat.4
    fs_insert(fs_insert(fs_insert(fs_insert(FiniteSet.empty[Point2[Real]], r01), r11), r21), r20).cardinality_is(Nat.4)
    r10 != r01
    r10 != r11
    r10 != r21
    r10 != r20
    fs_insert_contains_eq(fs_insert(fs_insert(fs_insert(FiniteSet.empty[Point2[Real]], r01), r11), r21), r20, r10)
    fs_insert(fs_insert(fs_insert(fs_insert(FiniteSet.empty[Point2[Real]], r01), r11), r21), r20).contains(r10) =
        (r10 = r20 or fs_insert(fs_insert(fs_insert(FiniteSet.empty[Point2[Real]], r01), r11), r21).contains(r10))
    fs_insert_contains_eq(fs_insert(fs_insert(FiniteSet.empty[Point2[Real]], r01), r11), r21, r10)
    fs_insert(fs_insert(fs_insert(FiniteSet.empty[Point2[Real]], r01), r11), r21).contains(r10) =
        (r10 = r21 or fs_insert(fs_insert(FiniteSet.empty[Point2[Real]], r01), r11).contains(r10))
    fs_insert_contains_eq(fs_insert(FiniteSet.empty[Point2[Real]], r01), r11, r10)
    fs_insert(fs_insert(FiniteSet.empty[Point2[Real]], r01), r11).contains(r10) =
        (r10 = r11 or fs_insert(FiniteSet.empty[Point2[Real]], r01).contains(r10))
    fs_insert_contains_eq(FiniteSet.empty[Point2[Real]], r01, r10)
    fs_insert(FiniteSet.empty[Point2[Real]], r01).contains(r10) =
        (r10 = r01 or FiniteSet.empty[Point2[Real]].contains(r10))
    finite_set_empty_contains_eq(r10)
    FiniteSet.empty[Point2[Real]].contains(r10) = false
    not fs_insert(fs_insert(fs_insert(fs_insert(FiniteSet.empty[Point2[Real]], r01), r11), r21), r20).contains(r10)
    finite_set_insert_cardinality_is_suc_of_not_contains(
        fs_insert(fs_insert(fs_insert(fs_insert(FiniteSet.empty[Point2[Real]], r01), r11), r21), r20), r10, Nat.4)
    fs_insert(fs_insert(fs_insert(fs_insert(fs_insert(FiniteSet.empty[Point2[Real]], r01), r11), r21), r20), r10).cardinality_is(Nat.4 + Nat.1)
    Nat.4 + Nat.1 = Nat.5
    fs_insert(fs_insert(fs_insert(fs_insert(fs_insert(FiniteSet.empty[Point2[Real]], r01), r11), r21), r20), r10).cardinality_is(Nat.5)
    r00 != r01
    r00 != r11
    r00 != r21
    r00 != r20
    r00 != r10
    fs_insert_contains_eq(fs_insert(fs_insert(fs_insert(fs_insert(FiniteSet.empty[Point2[Real]], r01), r11), r21), r20), r10, r00)
    fs_insert(fs_insert(fs_insert(fs_insert(fs_insert(FiniteSet.empty[Point2[Real]], r01), r11), r21), r20), r10).contains(r00) =
        (r00 = r10 or fs_insert(fs_insert(fs_insert(fs_insert(FiniteSet.empty[Point2[Real]], r01), r11), r21), r20).contains(r00))
    fs_insert_contains_eq(fs_insert(fs_insert(fs_insert(FiniteSet.empty[Point2[Real]], r01), r11), r21), r20, r00)
    fs_insert(fs_insert(fs_insert(fs_insert(FiniteSet.empty[Point2[Real]], r01), r11), r21), r20).contains(r00) =
        (r00 = r20 or fs_insert(fs_insert(fs_insert(FiniteSet.empty[Point2[Real]], r01), r11), r21).contains(r00))
    fs_insert_contains_eq(fs_insert(fs_insert(FiniteSet.empty[Point2[Real]], r01), r11), r21, r00)
    fs_insert(fs_insert(fs_insert(FiniteSet.empty[Point2[Real]], r01), r11), r21).contains(r00) =
        (r00 = r21 or fs_insert(fs_insert(FiniteSet.empty[Point2[Real]], r01), r11).contains(r00))
    fs_insert_contains_eq(fs_insert(FiniteSet.empty[Point2[Real]], r01), r11, r00)
    fs_insert(fs_insert(FiniteSet.empty[Point2[Real]], r01), r11).contains(r00) =
        (r00 = r11 or fs_insert(FiniteSet.empty[Point2[Real]], r01).contains(r00))
    fs_insert_contains_eq(FiniteSet.empty[Point2[Real]], r01, r00)
    fs_insert(FiniteSet.empty[Point2[Real]], r01).contains(r00) =
        (r00 = r01 or FiniteSet.empty[Point2[Real]].contains(r00))
    finite_set_empty_contains_eq(r00)
    FiniteSet.empty[Point2[Real]].contains(r00) = false
    not fs_insert(fs_insert(fs_insert(fs_insert(fs_insert(FiniteSet.empty[Point2[Real]], r01), r11), r21), r20), r10).contains(r00)
    finite_set_insert_cardinality_is_suc_of_not_contains(
        fs_insert(fs_insert(fs_insert(fs_insert(fs_insert(FiniteSet.empty[Point2[Real]], r01), r11), r21), r20), r10), r00, Nat.5)
    fs_insert(fs_insert(fs_insert(fs_insert(fs_insert(fs_insert(FiniteSet.empty[Point2[Real]], r01), r11), r21), r20), r10), r00).cardinality_is(Nat.5 + Nat.1)
    Nat.5 + Nat.1 = Nat.6
    fs_insert(fs_insert(fs_insert(fs_insert(fs_insert(fs_insert(FiniteSet.empty[Point2[Real]], r01), r11), r21), r20), r10), r00).cardinality_is(Nat.6)
    unit_rect_boundary = fs_insert(fs_insert(fs_insert(fs_insert(fs_insert(fs_insert(
        FiniteSet.empty[Point2[Real]], r01), r11), r21), r20), r10), r00)
    unit_rect_boundary.cardinality_is(Nat.6)
    fs_card_eq_of_cardinality_is(unit_rect_boundary, Nat.6)
    fs_card(unit_rect_boundary) = Nat.6
}

/// The rectangle has no interior lattice points.
theorem unit_rect_interior_card {
    fs_card(unit_rect_interior) = Nat.0
} by {
    fs_card_empty[Point2[Real]]
    fs_card(unit_rect_interior) = Nat.0
}

/// The rectangle has six boundary lattice points.
theorem unit_rect_boundary_lattice_count {
    lattice_point_count(unit_rect_boundary) = Nat.6
} by {
    lattice_point_count_of_lattice_set(unit_rect_boundary)
    forall(p: Point2[Real]) {
        if unit_rect_boundary.contains(p) {
            unit_rect_boundary_contains_eq(p)
            unit_rect_boundary.contains(p) =
                (p = r00 or (p = r10 or (p = r20 or (p = r21 or (p = r11 or p = r01)))))
            p = r00 or (p = r10 or (p = r20 or (p = r21 or (p = r11 or p = r01))))
            if p = r00 {
                is_lattice_point_r00
                is_lattice_point(p)
            } else {
                if p = r10 {
                    is_lattice_point_r10
                    is_lattice_point(p)
                } else {
                    if p = r20 {
                        is_lattice_point_r20
                        is_lattice_point(p)
                    } else {
                        if p = r21 {
                            is_lattice_point_r21
                            is_lattice_point(p)
                        } else {
                            if p = r11 {
                                is_lattice_point_r11
                                is_lattice_point(p)
                            } else {
                                p = r01
                                is_lattice_point_r01
                                is_lattice_point(p)
                            }
                        }
                    }
                }
            }
        }
    }
    lattice_point_count(unit_rect_boundary) = fs_card(unit_rect_boundary)
    unit_rect_boundary_card
    lattice_point_count(unit_rect_boundary) = Nat.6
}

/// The rectangle has no interior lattice points.
theorem unit_rect_interior_lattice_count {
    lattice_point_count(unit_rect_interior) = Nat.0
} by {
    unit_rect_interior = FiniteSet.empty[Point2[Real]]
    lattice_point_count_empty
    lattice_point_count(unit_rect_interior) = Nat.0
}

/// Pick's theorem for the 1-by-2 rectangle:
/// `Area = 2 = I + B/2 - 1 = 0 + 6/2 - 1`.
theorem pick_unit_rect {
    is_simple_lattice_polygon(unit_rect) implies
    polygon_area(unit_rect) =
        from_nat[Real](lattice_point_count(unit_rect_interior)) +
        from_nat[Real](lattice_point_count(unit_rect_boundary)) / real_two - Real.1
} by {
    if is_simple_lattice_polygon(unit_rect) {
        unit_rect_area_is_two
        polygon_area(unit_rect) = real_two
        unit_rect_interior_lattice_count
        lattice_point_count(unit_rect_interior) = Nat.0
        unit_rect_boundary_lattice_count
        lattice_point_count(unit_rect_boundary) = Nat.6
        from_nat_zero[Real]
        from_nat[Real](Nat.0) = Real.0
        from_nat[Real](lattice_point_count(unit_rect_interior)) = Real.0
        from_nat_six
        from_nat[Real](Nat.6) = real_six
        from_nat[Real](lattice_point_count(unit_rect_boundary)) = real_six
        from_nat[Real](lattice_point_count(unit_rect_boundary)) / real_two = real_six / real_two
        real_six_div_two
        real_six / real_two = real_three
        from_nat[Real](lattice_point_count(unit_rect_boundary)) / real_two = real_three
        from_nat[Real](lattice_point_count(unit_rect_interior)) +
            from_nat[Real](lattice_point_count(unit_rect_boundary)) / real_two - Real.1 =
            Real.0 + real_three - Real.1
        Real.0 + real_three - Real.1 = real_two
        polygon_area(unit_rect) =
            from_nat[Real](lattice_point_count(unit_rect_interior)) +
            from_nat[Real](lattice_point_count(unit_rect_boundary)) / real_two - Real.1
    }
}

/// Pick's theorem for the 1-by-2 rectangle in the doubled-area form:
/// `2 * Area = 2I + B - 2 = 0 + 6 - 2`.
theorem pick_unit_rect_doubled {
    is_simple_lattice_polygon(unit_rect) implies
    point2_polygon_area2(unit_rect) =
        from_nat[Real](lattice_point_count(unit_rect_interior) +
            lattice_point_count(unit_rect_interior) +
            lattice_point_count(unit_rect_boundary)) - real_two
} by {
    if is_simple_lattice_polygon(unit_rect) {
        unit_rect_area2_is_four
        point2_polygon_area2(unit_rect) = real_four
        unit_rect_interior_lattice_count
        lattice_point_count(unit_rect_interior) = Nat.0
        unit_rect_boundary_lattice_count
        lattice_point_count(unit_rect_boundary) = Nat.6
        lattice_point_count(unit_rect_interior) +
            lattice_point_count(unit_rect_interior) +
            lattice_point_count(unit_rect_boundary) = Nat.6
        from_nat_six
        from_nat[Real](Nat.6) = real_six
        from_nat[Real](lattice_point_count(unit_rect_interior) +
            lattice_point_count(unit_rect_interior) +
            lattice_point_count(unit_rect_boundary)) = real_six
        from_nat[Real](lattice_point_count(unit_rect_interior) +
            lattice_point_count(unit_rect_interior) +
            lattice_point_count(unit_rect_boundary)) - real_two = real_six - real_two
        real_six = real_four + real_two
        real_six - real_two = (real_four + real_two) - real_two
        (real_four + real_two) - real_two = real_four
        real_six - real_two = real_four
        point2_polygon_area2(unit_rect) =
            from_nat[Real](lattice_point_count(unit_rect_interior) +
                lattice_point_count(unit_rect_interior) +
                lattice_point_count(unit_rect_boundary)) - real_two
    }
}

// ---------------------------------------------------------------------------
// Case 3: the 3-4-5 right triangle with legs on the axes.
//
//   t03
//   | \
//   t02  \
//   |     \
//   t01      \
//   |          \
//   t00--t10--t20--t30--t40
//
// Area 6, I = 3 (the points (1,1), (1,2), (2,1)), B = 8
// (five points on the bottom edge and three on the left edge).
// Pick: 3 + 8/2 - 1 = 6.
// ---------------------------------------------------------------------------

/// The real number fourteen.
let real_fourteen: Real = real_twelve + real_two

/// Twelve embeds as the real number twelve.
theorem from_nat_twelve {
    from_nat[Real](Nat.12) = real_twelve
} by {
    from_nat_mul[Real](Nat.4, Nat.3)
    from_nat[Real](Nat.4) * from_nat[Real](Nat.3) = from_nat[Real](Nat.4 * Nat.3)
    Nat.4 * Nat.3 = Nat.12
    from_nat[Real](Nat.4) * from_nat[Real](Nat.3) = from_nat[Real](Nat.12)
    from_nat_four
    from_nat_three
    from_nat[Real](Nat.4) * from_nat[Real](Nat.3) = real_four * real_three
    real_four * real_three = real_twelve
    from_nat[Real](Nat.12) = real_twelve
}

/// Fourteen embeds as the real number fourteen.
theorem from_nat_fourteen {
    from_nat[Real](Nat.14) = real_fourteen
} by {
    from_nat_add[Real](Nat.12, Nat.2)
    from_nat[Real](Nat.12 + Nat.2) = from_nat[Real](Nat.12) + from_nat[Real](Nat.2)
    Nat.12 + Nat.1 = Nat.13
    Nat.13 + Nat.1 = Nat.14
    Nat.12 + Nat.2 = Nat.14
    from_nat[Real](Nat.14) = from_nat[Real](Nat.12) + from_nat[Real](Nat.2)
    from_nat_twelve
    from_nat_two
    from_nat[Real](Nat.12) + from_nat[Real](Nat.2) = real_twelve + real_two
    real_twelve + real_two = real_fourteen
    from_nat[Real](Nat.14) = real_fourteen
}

/// The origin vertex of the triangle.
let t00: Point2[Real] = Point2.new(Real.0, Real.0)

/// The first bottom-edge lattice point of the triangle.
let t10: Point2[Real] = Point2.new(Real.1, Real.0)

/// The second bottom-edge lattice point of the triangle.
let t20: Point2[Real] = Point2.new(real_two, Real.0)

/// The third bottom-edge lattice point of the triangle.
let t30: Point2[Real] = Point2.new(real_three, Real.0)

/// The far bottom vertex of the triangle.
let t40: Point2[Real] = Point2.new(real_four, Real.0)

/// The first left-edge lattice point of the triangle.
let t01: Point2[Real] = Point2.new(Real.0, Real.1)

/// The second left-edge lattice point of the triangle.
let t02: Point2[Real] = Point2.new(Real.0, real_two)

/// The top vertex of the triangle.
let t03: Point2[Real] = Point2.new(Real.0, real_three)

/// The interior lattice point (1,1) of the triangle.
let p11: Point2[Real] = Point2.new(Real.1, Real.1)

/// The interior lattice point (1,2) of the triangle.
let p12: Point2[Real] = Point2.new(Real.1, real_two)

/// The interior lattice point (2,1) of the triangle.
let p21: Point2[Real] = Point2.new(real_two, Real.1)

/// The 3-4-5 triangle as a counter-clockwise vertex list.
let right_triangle: List[Point2[Real]] =
    List.cons(t00, List.cons(t40, List.cons(t03, List.nil[Point2[Real]])))

/// The bottom-edge lattice points of the triangle.
let triangle_bottom: FiniteSet[Point2[Real]] =
    fs_insert(fs_insert(fs_insert(fs_insert(fs_insert(
        FiniteSet.empty[Point2[Real]], t40), t30), t20), t10), t00)

/// The left-edge lattice points of the triangle.
let triangle_left: FiniteSet[Point2[Real]] =
    fs_insert(fs_insert(fs_insert(
        FiniteSet.empty[Point2[Real]], t03), t02), t01)

/// The boundary lattice points of the triangle: the bottom and left edges.
let triangle_boundary: FiniteSet[Point2[Real]] =
    fs_union(triangle_bottom, triangle_left)

/// The interior lattice points of the triangle.
let triangle_interior: FiniteSet[Point2[Real]] =
    fs_insert(fs_insert(fs_insert(
        FiniteSet.empty[Point2[Real]], p21), p12), p11)

/// The bottom-edge points of the triangle are lattice points.
theorem is_lattice_point_t00 {
    is_lattice_point(t00)
} by {
    t00.x = Real.0
    Real.from_int(Int.0) = Real.0
    t00.x = Real.from_int(Int.0)
    t00.y = Real.0
    t00.y = Real.from_int(Int.0)
    t00.x = Real.from_int(Int.0) and t00.y = Real.from_int(Int.0)
    is_lattice_point_of_coords(t00, Int.0, Int.0)
    is_lattice_point(t00)
}

/// The bottom-edge points of the triangle are lattice points.
theorem is_lattice_point_t10 {
    is_lattice_point(t10)
} by {
    t10.x = Real.1
    Real.from_int(Int.1) = Real.1
    t10.x = Real.from_int(Int.1)
    t10.y = Real.0
    t10.y = Real.from_int(Int.0)
    t10.x = Real.from_int(Int.1) and t10.y = Real.from_int(Int.0)
    is_lattice_point_of_coords(t10, Int.1, Int.0)
    is_lattice_point(t10)
}

/// The bottom-edge points of the triangle are lattice points.
theorem is_lattice_point_t20 {
    is_lattice_point(t20)
} by {
    t20.x = real_two
    real_from_int_from_nat(Nat.2)
    from_nat_two
    Real.from_int(Int.from_nat(Nat.2)) = real_two
    t20.x = Real.from_int(Int.from_nat(Nat.2))
    t20.y = Real.0
    Real.from_int(Int.0) = Real.0
    t20.y = Real.from_int(Int.0)
    t20.x = Real.from_int(Int.from_nat(Nat.2)) and t20.y = Real.from_int(Int.0)
    is_lattice_point_of_coords(t20, Int.from_nat(Nat.2), Int.0)
    is_lattice_point(t20)
}

/// The bottom-edge points of the triangle are lattice points.
theorem is_lattice_point_t30 {
    is_lattice_point(t30)
} by {
    t30.x = real_three
    real_from_int_from_nat(Nat.3)
    from_nat_three
    Real.from_int(Int.from_nat(Nat.3)) = real_three
    t30.x = Real.from_int(Int.from_nat(Nat.3))
    t30.y = Real.0
    Real.from_int(Int.0) = Real.0
    t30.y = Real.from_int(Int.0)
    t30.x = Real.from_int(Int.from_nat(Nat.3)) and t30.y = Real.from_int(Int.0)
    is_lattice_point_of_coords(t30, Int.from_nat(Nat.3), Int.0)
    is_lattice_point(t30)
}

/// The bottom-edge points of the triangle are lattice points.
theorem is_lattice_point_t40 {
    is_lattice_point(t40)
} by {
    t40.x = real_four
    real_from_int_from_nat(Nat.4)
    from_nat_four
    Real.from_int(Int.from_nat(Nat.4)) = real_four
    t40.x = Real.from_int(Int.from_nat(Nat.4))
    t40.y = Real.0
    Real.from_int(Int.0) = Real.0
    t40.y = Real.from_int(Int.0)
    t40.x = Real.from_int(Int.from_nat(Nat.4)) and t40.y = Real.from_int(Int.0)
    is_lattice_point_of_coords(t40, Int.from_nat(Nat.4), Int.0)
    is_lattice_point(t40)
}

/// The left-edge points of the triangle are lattice points.
theorem is_lattice_point_t01 {
    is_lattice_point(t01)
} by {
    t01.x = Real.0
    Real.from_int(Int.0) = Real.0
    t01.x = Real.from_int(Int.0)
    t01.y = Real.1
    Real.from_int(Int.1) = Real.1
    t01.y = Real.from_int(Int.1)
    t01.x = Real.from_int(Int.0) and t01.y = Real.from_int(Int.1)
    is_lattice_point_of_coords(t01, Int.0, Int.1)
    is_lattice_point(t01)
}

/// The left-edge points of the triangle are lattice points.
theorem is_lattice_point_t02 {
    is_lattice_point(t02)
} by {
    t02.x = Real.0
    Real.from_int(Int.0) = Real.0
    t02.x = Real.from_int(Int.0)
    t02.y = real_two
    real_from_int_from_nat(Nat.2)
    from_nat_two
    Real.from_int(Int.from_nat(Nat.2)) = real_two
    t02.y = Real.from_int(Int.from_nat(Nat.2))
    t02.x = Real.from_int(Int.0) and t02.y = Real.from_int(Int.from_nat(Nat.2))
    is_lattice_point_of_coords(t02, Int.0, Int.from_nat(Nat.2))
    is_lattice_point(t02)
}

/// The left-edge points of the triangle are lattice points.
theorem is_lattice_point_t03 {
    is_lattice_point(t03)
} by {
    t03.x = Real.0
    Real.from_int(Int.0) = Real.0
    t03.x = Real.from_int(Int.0)
    t03.y = real_three
    real_from_int_from_nat(Nat.3)
    from_nat_three
    Real.from_int(Int.from_nat(Nat.3)) = real_three
    t03.y = Real.from_int(Int.from_nat(Nat.3))
    t03.x = Real.from_int(Int.0) and t03.y = Real.from_int(Int.from_nat(Nat.3))
    is_lattice_point_of_coords(t03, Int.0, Int.from_nat(Nat.3))
    is_lattice_point(t03)
}

/// The interior points of the triangle are lattice points.
theorem is_lattice_point_p11 {
    is_lattice_point(p11)
} by {
    p11.x = Real.1
    Real.from_int(Int.1) = Real.1
    p11.x = Real.from_int(Int.1)
    p11.y = Real.1
    p11.y = Real.from_int(Int.1)
    p11.x = Real.from_int(Int.1) and p11.y = Real.from_int(Int.1)
    is_lattice_point_of_coords(p11, Int.1, Int.1)
    is_lattice_point(p11)
}

/// The interior points of the triangle are lattice points.
theorem is_lattice_point_p12 {
    is_lattice_point(p12)
} by {
    p12.x = Real.1
    Real.from_int(Int.1) = Real.1
    p12.x = Real.from_int(Int.1)
    p12.y = real_two
    real_from_int_from_nat(Nat.2)
    from_nat_two
    Real.from_int(Int.from_nat(Nat.2)) = real_two
    p12.y = Real.from_int(Int.from_nat(Nat.2))
    p12.x = Real.from_int(Int.1) and p12.y = Real.from_int(Int.from_nat(Nat.2))
    is_lattice_point_of_coords(p12, Int.1, Int.from_nat(Nat.2))
    is_lattice_point(p12)
}

/// The interior points of the triangle are lattice points.
theorem is_lattice_point_p21 {
    is_lattice_point(p21)
} by {
    p21.x = real_two
    real_from_int_from_nat(Nat.2)
    from_nat_two
    Real.from_int(Int.from_nat(Nat.2)) = real_two
    p21.x = Real.from_int(Int.from_nat(Nat.2))
    p21.y = Real.1
    Real.from_int(Int.1) = Real.1
    p21.y = Real.from_int(Int.1)
    p21.x = Real.from_int(Int.from_nat(Nat.2)) and p21.y = Real.from_int(Int.1)
    is_lattice_point_of_coords(p21, Int.from_nat(Nat.2), Int.1)
    is_lattice_point(p21)
}

/// The triangle is a lattice polygon.
theorem is_lattice_polygon_right_triangle {
    is_lattice_polygon(right_triangle)
} by {
    is_lattice_polygon_nil
    is_lattice_polygon(List.nil[Point2[Real]])
    is_lattice_polygon_intro(t03, List.nil[Point2[Real]])
    is_lattice_point(t03) and is_lattice_polygon(List.nil[Point2[Real]])
    is_lattice_polygon(List.cons(t03, List.nil[Point2[Real]]))
    is_lattice_polygon_intro(t40, List.cons(t03, List.nil[Point2[Real]]))
    is_lattice_point(t40) and is_lattice_polygon(List.cons(t03, List.nil[Point2[Real]]))
    is_lattice_polygon(List.cons(t40, List.cons(t03, List.nil[Point2[Real]])))
    is_lattice_polygon_intro(t00, List.cons(t40, List.cons(t03, List.nil[Point2[Real]])))
    is_lattice_point(t00) and is_lattice_polygon(List.cons(t40, List.cons(t03, List.nil[Point2[Real]])))
    is_lattice_polygon(right_triangle)
}

/// The doubled shoelace area of the triangle is twelve.
theorem right_triangle_area2_is_twelve {
    point2_polygon_area2(right_triangle) = real_twelve
} by {
    point2_polygon_area2_triple(t00, t40, t03)
    point2_polygon_area2(right_triangle) = t00.triangle_area2(t40, t03)
    point2_triangle_area2_eq_orientation(t00, t40, t03)
    point2_orientation_coordinate_formula(t00, t40, t03)
    point2_new_x(Real.0, Real.0)
    point2_new_y(Real.0, Real.0)
    point2_new_x(real_four, Real.0)
    point2_new_y(real_four, Real.0)
    point2_new_x(Real.0, real_three)
    point2_new_y(Real.0, real_three)
    t00.x = Real.0
    t00.y = Real.0
    t40.x = real_four
    t40.y = Real.0
    t03.x = Real.0
    t03.y = real_three
    t00.orientation(t40, t03) = (real_four - Real.0) * (real_three - Real.0) - (Real.0 - Real.0) * (Real.0 - Real.0)
    (real_four - Real.0) * (real_three - Real.0) = real_four * real_three
    (Real.0 - Real.0) * (Real.0 - Real.0) = Real.0
    (real_four - Real.0) * (real_three - Real.0) - (Real.0 - Real.0) * (Real.0 - Real.0) =
        real_four * real_three - Real.0
    real_four * real_three = real_twelve
    real_four * real_three - Real.0 = real_twelve
    (real_four - Real.0) * (real_three - Real.0) - (Real.0 - Real.0) * (Real.0 - Real.0) = real_twelve
    t00.triangle_area2(t40, t03) = real_twelve
    point2_polygon_area2(right_triangle) = real_twelve
}

/// The triangle winds counter-clockwise.
theorem right_triangle_ccw {
    point2_polygon_ccw(right_triangle)
} by {
    right_triangle_area2_is_twelve
    point2_polygon_area2(right_triangle) = real_twelve
    real_lt_0_4
    Real.0 < real_four
    gt_zero_imp_pos(real_four)
    real_four.is_positive
    real_lt_0_3
    Real.0 < real_three
    gt_zero_imp_pos(real_three)
    real_three.is_positive
    mul_pos_pos(real_four, real_three)
    (real_four * real_three).is_positive
    real_four * real_three = real_twelve
    real_twelve.is_positive
    pos_gt_zero(real_twelve)
    Real.0 < real_twelve
    point2_polygon_area2(right_triangle) > Real.0
    point2_polygon_ccw(right_triangle) = (point2_polygon_area2(right_triangle) > Real.0)
    point2_polygon_ccw(right_triangle)
}

/// The triangle is a simple lattice polygon.
theorem is_simple_lattice_polygon_right_triangle {
    is_simple_lattice_polygon(right_triangle)
} by {
    is_lattice_polygon_right_triangle
    right_triangle_ccw
    is_simple_lattice_polygon(right_triangle) =
        (is_lattice_polygon(right_triangle) and point2_polygon_ccw(right_triangle))
    is_lattice_polygon(right_triangle) and point2_polygon_ccw(right_triangle)
    is_simple_lattice_polygon(right_triangle)
}

/// Four times two is eight.
theorem real_four_times_two_is_eight {
    real_four * real_two = real_eight
} by {
    real_four = real_two + real_two
    (real_two + real_two) * real_two = real_two * real_two + real_two * real_two
    real_two * real_two = real_four
    real_two * real_two + real_two * real_two = real_four + real_four
    real_eight = real_six + real_two
    real_six = real_four + real_two
    real_eight = real_four + real_two + real_two
    real_four + real_two + real_two = real_four + (real_two + real_two)
    real_two + real_two = real_four
    real_four + (real_two + real_two) = real_four + real_four
    real_four + real_four = real_eight
    real_four * real_two = real_eight
}

/// Four times three is six plus six.
theorem real_four_times_three_is_six_plus_six {
    real_four * real_three = real_six + real_six
} by {
    real_three = real_two + Real.1
    real_four * real_three = real_four * (real_two + Real.1)
    real_four * (real_two + Real.1) = real_four * real_two + real_four * Real.1
    real_four_times_two_is_eight
    real_four * real_two = real_eight
    real_four * Real.1 = real_four
    real_four * real_two + real_four * Real.1 = real_eight + real_four
    real_eight + real_four = real_six + real_six
    real_four * real_three = real_six + real_six
}

/// Six plus six is twelve.
theorem real_six_plus_six_is_twelve {
    real_six + real_six = real_twelve
} by {
    real_four_times_three_is_six_plus_six
    real_six + real_six = real_four * real_three
    real_four * real_three = real_twelve
    real_six + real_six = real_twelve
}

/// Three plus three is six.
theorem real_three_plus_three_is_six {
    real_three + real_three = real_six
} by {
    real_three = real_two + Real.1
    real_three + real_three = (real_two + Real.1) + (real_two + Real.1)
    (real_two + Real.1) + (real_two + Real.1) = (real_two + real_two) + (Real.1 + Real.1)
    real_two + real_two = real_four
    Real.1 + Real.1 = real_two
    (real_two + real_two) + (Real.1 + Real.1) = real_four + real_two
    real_four + real_two = real_six
    real_three + real_three = real_six
}

/// Six plus eight is fourteen.
theorem real_six_plus_eight_is_fourteen {
    real_six + real_eight = real_fourteen
} by {
    real_eight = real_six + real_two
    real_six + real_eight = real_six + (real_six + real_two)
    real_six + (real_six + real_two) = (real_six + real_six) + real_two
    real_six_plus_six_is_twelve
    real_six + real_six = real_twelve
    (real_six + real_six) + real_two = real_twelve + real_two
    real_twelve + real_two = real_fourteen
    real_six + real_eight = real_fourteen
}

/// The area of the triangle is six.
theorem right_triangle_area_is_six {
    polygon_area(right_triangle) = real_six
} by {
    right_triangle_area2_is_twelve
    point2_polygon_area2(right_triangle) = real_twelve
    polygon_area(right_triangle) = point2_polygon_area2(right_triangle) / real_two
    polygon_area(right_triangle) = real_twelve / real_two
    real_six_plus_six_is_twelve
    real_twelve = real_six + real_six
    real_twelve / real_two = (real_six + real_six) / real_two
    div_add_distrib_local(real_six, real_six, real_two)
    real_two != Real.0
    (real_six + real_six) / real_two = real_six / real_two + real_six / real_two
    real_six = real_four + real_two
    real_six / real_two = (real_four + real_two) / real_two
    div_add_distrib_local(real_four, real_two, real_two)
    real_two != Real.0
    (real_four + real_two) / real_two = real_four / real_two + real_two / real_two
    real_four_div_two
    real_four / real_two = real_two
    real_two_div_two
    real_two / real_two = Real.1
    real_four / real_two + real_two / real_two = real_two + Real.1
    real_two + Real.1 = real_three
    real_six / real_two = real_three
    real_six / real_two + real_six / real_two = real_three + real_three
    real_three_plus_three_is_six
    real_three + real_three = real_six
    real_twelve / real_two = real_six
    polygon_area(right_triangle) = real_six
}

/// The bottom-edge lattice points of the triangle are distinct.
theorem triangle_bottom_distinct {
    t00 != t10 and t10 != t20 and t20 != t30 and t30 != t40 and
    t00 != t20 and t10 != t30 and t20 != t40 and
    t00 != t30 and t10 != t40 and
    t00 != t40
} by {
    point2_ne_of_x_ne(t00, t10)
    t00.x = Real.0
    t10.x = Real.1
    real_ne_zero_one
    Real.0 != Real.1
    t00.x != t10.x
    t00 != t10
    point2_ne_of_x_ne(t10, t20)
    t10.x = Real.1
    t20.x = real_two
    real_ne_1_2
    Real.1 != real_two
    t10.x != t20.x
    t10 != t20
    point2_ne_of_x_ne(t20, t30)
    t20.x = real_two
    t30.x = real_three
    real_ne_2_3
    real_two != real_three
    t20.x != t30.x
    t20 != t30
    point2_ne_of_x_ne(t30, t40)
    t30.x = real_three
    t40.x = real_four
    real_ne_3_4
    real_three != real_four
    t30.x != t40.x
    t30 != t40
    point2_ne_of_x_ne(t00, t20)
    t00.x = Real.0
    t20.x = real_two
    real_ne_0_2
    Real.0 != real_two
    t00.x != t20.x
    t00 != t20
    point2_ne_of_x_ne(t10, t30)
    t10.x = Real.1
    t30.x = real_three
    real_ne_1_3
    Real.1 != real_three
    t10.x != t30.x
    t10 != t30
    point2_ne_of_x_ne(t20, t40)
    t20.x = real_two
    t40.x = real_four
    real_ne_2_4
    real_two != real_four
    t20.x != t40.x
    t20 != t40
    point2_ne_of_x_ne(t00, t30)
    t00.x = Real.0
    t30.x = real_three
    real_ne_0_3
    Real.0 != real_three
    t00.x != t30.x
    t00 != t30
    point2_ne_of_x_ne(t10, t40)
    t10.x = Real.1
    t40.x = real_four
    real_ne_1_4
    Real.1 != real_four
    t10.x != t40.x
    t10 != t40
    point2_ne_of_x_ne(t00, t40)
    t00.x = Real.0
    t40.x = real_four
    real_ne_0_4
    Real.0 != real_four
    t00.x != t40.x
    t00 != t40
    t00 != t10 and t10 != t20 and t20 != t30 and t30 != t40 and
        t00 != t20 and t10 != t30 and t20 != t40 and
        t00 != t30 and t10 != t40 and
        t00 != t40
}

/// The bottom edge of the triangle has five lattice points.
theorem triangle_bottom_card {
    fs_card(triangle_bottom) = Nat.5
} by {
    fs_card_empty[Point2[Real]]
    fs_card(FiniteSet.empty[Point2[Real]]) = Nat.0
    fs_card_cardinality_is(FiniteSet.empty[Point2[Real]])
    FiniteSet.empty[Point2[Real]].cardinality_is(fs_card(FiniteSet.empty[Point2[Real]]))
    FiniteSet.empty[Point2[Real]].cardinality_is(Nat.0)
    finite_set_empty_contains_eq(t40)
    not FiniteSet.empty[Point2[Real]].contains(t40)
    finite_set_insert_cardinality_is_suc_of_not_contains(FiniteSet.empty[Point2[Real]], t40, Nat.0)
    fs_insert(FiniteSet.empty[Point2[Real]], t40).cardinality_is(Nat.1)
    triangle_bottom_distinct
    t30 != t40
    fs_insert_contains_eq(FiniteSet.empty[Point2[Real]], t40, t30)
    fs_insert(FiniteSet.empty[Point2[Real]], t40).contains(t30) =
        (t30 = t40 or FiniteSet.empty[Point2[Real]].contains(t30))
    finite_set_empty_contains_eq(t30)
    FiniteSet.empty[Point2[Real]].contains(t30) = false
    not fs_insert(FiniteSet.empty[Point2[Real]], t40).contains(t30)
    finite_set_insert_cardinality_is_suc_of_not_contains(
        fs_insert(FiniteSet.empty[Point2[Real]], t40), t30, Nat.1)
    fs_insert(fs_insert(FiniteSet.empty[Point2[Real]], t40), t30).cardinality_is(Nat.2)
    t20 != t40
    t20 != t30
    fs_insert_contains_eq(fs_insert(FiniteSet.empty[Point2[Real]], t40), t30, t20)
    fs_insert(fs_insert(FiniteSet.empty[Point2[Real]], t40), t30).contains(t20) =
        (t20 = t30 or fs_insert(FiniteSet.empty[Point2[Real]], t40).contains(t20))
    fs_insert_contains_eq(FiniteSet.empty[Point2[Real]], t40, t20)
    fs_insert(FiniteSet.empty[Point2[Real]], t40).contains(t20) =
        (t20 = t40 or FiniteSet.empty[Point2[Real]].contains(t20))
    finite_set_empty_contains_eq(t20)
    FiniteSet.empty[Point2[Real]].contains(t20) = false
    not fs_insert(fs_insert(FiniteSet.empty[Point2[Real]], t40), t30).contains(t20)
    finite_set_insert_cardinality_is_suc_of_not_contains(
        fs_insert(fs_insert(FiniteSet.empty[Point2[Real]], t40), t30), t20, Nat.2)
    fs_insert(fs_insert(fs_insert(FiniteSet.empty[Point2[Real]], t40), t30), t20).cardinality_is(Nat.3)
    t10 != t40
    t10 != t30
    t10 != t20
    fs_insert_contains_eq(fs_insert(fs_insert(FiniteSet.empty[Point2[Real]], t40), t30), t20, t10)
    fs_insert(fs_insert(fs_insert(FiniteSet.empty[Point2[Real]], t40), t30), t20).contains(t10) =
        (t10 = t20 or fs_insert(fs_insert(FiniteSet.empty[Point2[Real]], t40), t30).contains(t10))
    fs_insert_contains_eq(fs_insert(FiniteSet.empty[Point2[Real]], t40), t30, t10)
    fs_insert(fs_insert(FiniteSet.empty[Point2[Real]], t40), t30).contains(t10) =
        (t10 = t30 or fs_insert(FiniteSet.empty[Point2[Real]], t40).contains(t10))
    fs_insert_contains_eq(FiniteSet.empty[Point2[Real]], t40, t10)
    fs_insert(FiniteSet.empty[Point2[Real]], t40).contains(t10) =
        (t10 = t40 or FiniteSet.empty[Point2[Real]].contains(t10))
    finite_set_empty_contains_eq(t10)
    FiniteSet.empty[Point2[Real]].contains(t10) = false
    not fs_insert(fs_insert(fs_insert(FiniteSet.empty[Point2[Real]], t40), t30), t20).contains(t10)
    finite_set_insert_cardinality_is_suc_of_not_contains(
        fs_insert(fs_insert(fs_insert(FiniteSet.empty[Point2[Real]], t40), t30), t20), t10, Nat.3)
    fs_insert(fs_insert(fs_insert(fs_insert(FiniteSet.empty[Point2[Real]], t40), t30), t20), t10).cardinality_is(Nat.4)
    t00 != t40
    t00 != t30
    t00 != t20
    t00 != t10
    fs_insert_contains_eq(fs_insert(fs_insert(fs_insert(FiniteSet.empty[Point2[Real]], t40), t30), t20), t10, t00)
    fs_insert(fs_insert(fs_insert(fs_insert(FiniteSet.empty[Point2[Real]], t40), t30), t20), t10).contains(t00) =
        (t00 = t10 or fs_insert(fs_insert(fs_insert(FiniteSet.empty[Point2[Real]], t40), t30), t20).contains(t00))
    fs_insert_contains_eq(fs_insert(fs_insert(FiniteSet.empty[Point2[Real]], t40), t30), t20, t00)
    fs_insert(fs_insert(fs_insert(FiniteSet.empty[Point2[Real]], t40), t30), t20).contains(t00) =
        (t00 = t20 or fs_insert(fs_insert(FiniteSet.empty[Point2[Real]], t40), t30).contains(t00))
    fs_insert_contains_eq(fs_insert(FiniteSet.empty[Point2[Real]], t40), t30, t00)
    fs_insert(fs_insert(FiniteSet.empty[Point2[Real]], t40), t30).contains(t00) =
        (t00 = t30 or fs_insert(FiniteSet.empty[Point2[Real]], t40).contains(t00))
    fs_insert_contains_eq(FiniteSet.empty[Point2[Real]], t40, t00)
    fs_insert(FiniteSet.empty[Point2[Real]], t40).contains(t00) =
        (t00 = t40 or FiniteSet.empty[Point2[Real]].contains(t00))
    finite_set_empty_contains_eq(t00)
    FiniteSet.empty[Point2[Real]].contains(t00) = false
    not fs_insert(fs_insert(fs_insert(fs_insert(FiniteSet.empty[Point2[Real]], t40), t30), t20), t10).contains(t00)
    finite_set_insert_cardinality_is_suc_of_not_contains(
        fs_insert(fs_insert(fs_insert(fs_insert(FiniteSet.empty[Point2[Real]], t40), t30), t20), t10), t00, Nat.4)
    fs_insert(fs_insert(fs_insert(fs_insert(fs_insert(FiniteSet.empty[Point2[Real]], t40), t30), t20), t10), t00).cardinality_is(Nat.4 + Nat.1)
    Nat.4 + Nat.1 = Nat.5
    fs_insert(fs_insert(fs_insert(fs_insert(fs_insert(FiniteSet.empty[Point2[Real]], t40), t30), t20), t10), t00).cardinality_is(Nat.5)
    triangle_bottom = fs_insert(fs_insert(fs_insert(fs_insert(fs_insert(
        FiniteSet.empty[Point2[Real]], t40), t30), t20), t10), t00)
    triangle_bottom.cardinality_is(Nat.5)
    fs_card_eq_of_cardinality_is(triangle_bottom, Nat.5)
    fs_card(triangle_bottom) = Nat.5
}

/// The bottom edge of the triangle contains exactly its five grid points.
theorem triangle_bottom_contains_eq(x: Point2[Real]) {
    triangle_bottom.contains(x) =
        (x = t00 or (x = t10 or (x = t20 or (x = t30 or x = t40))))
} by {
    fs_insert_contains_eq(fs_insert(fs_insert(fs_insert(fs_insert(
        FiniteSet.empty[Point2[Real]], t40), t30), t20), t10), t00, x)
    triangle_bottom.contains(x) =
        (x = t00 or fs_insert(fs_insert(fs_insert(fs_insert(
            FiniteSet.empty[Point2[Real]], t40), t30), t20), t10).contains(x))
    fs_insert_contains_eq(fs_insert(fs_insert(fs_insert(
        FiniteSet.empty[Point2[Real]], t40), t30), t20), t10, x)
    fs_insert(fs_insert(fs_insert(fs_insert(
        FiniteSet.empty[Point2[Real]], t40), t30), t20), t10).contains(x) =
        (x = t10 or fs_insert(fs_insert(fs_insert(
            FiniteSet.empty[Point2[Real]], t40), t30), t20).contains(x))
    fs_insert_contains_eq(fs_insert(fs_insert(
        FiniteSet.empty[Point2[Real]], t40), t30), t20, x)
    fs_insert(fs_insert(fs_insert(
        FiniteSet.empty[Point2[Real]], t40), t30), t20).contains(x) =
        (x = t20 or fs_insert(fs_insert(
            FiniteSet.empty[Point2[Real]], t40), t30).contains(x))
    fs_insert_contains_eq(fs_insert(FiniteSet.empty[Point2[Real]], t40), t30, x)
    fs_insert(fs_insert(FiniteSet.empty[Point2[Real]], t40), t30).contains(x) =
        (x = t30 or fs_insert(FiniteSet.empty[Point2[Real]], t40).contains(x))
    fs_insert_contains_eq(FiniteSet.empty[Point2[Real]], t40, x)
    fs_insert(FiniteSet.empty[Point2[Real]], t40).contains(x) =
        (x = t40 or FiniteSet.empty[Point2[Real]].contains(x))
    finite_set_empty_contains_eq(x)
    FiniteSet.empty[Point2[Real]].contains(x) = false
    triangle_bottom.contains(x) =
        (x = t00 or (x = t10 or (x = t20 or (x = t30 or (x = t40 or false)))))
    or_false(x = t40)
    (x = t40 or false) = (x = t40)
    triangle_bottom.contains(x) =
        (x = t00 or (x = t10 or (x = t20 or (x = t30 or x = t40))))
}

/// Every point of the bottom edge has y-coordinate zero.
theorem triangle_bottom_member_y_zero(x: Point2[Real]) {
    triangle_bottom.contains(x) implies x.y = Real.0
} by {
    if triangle_bottom.contains(x) {
        triangle_bottom_contains_eq(x)
        triangle_bottom.contains(x) =
            (x = t00 or (x = t10 or (x = t20 or (x = t30 or x = t40))))
        x = t00 or (x = t10 or (x = t20 or (x = t30 or x = t40)))
        if x = t00 {
            x = t00
            t00.y = Real.0
            x.y = Real.0
        } else {
            if x = t10 {
                x = t10
                t10.y = Real.0
                x.y = Real.0
            } else {
                if x = t20 {
                    x = t20
                    t20.y = Real.0
                    x.y = Real.0
                } else {
                    if x = t30 {
                        x = t30
                        t30.y = Real.0
                        x.y = Real.0
                    } else {
                        x = t40
                        t40.y = Real.0
                        x.y = Real.0
                    }
                }
            }
        }
    }
}

/// The left-edge lattice points of the triangle are distinct.
theorem triangle_left_distinct {
    t01 != t02 and t02 != t03 and t01 != t03
} by {
    point2_ne_of_y_ne(t01, t02)
    t01.y = Real.1
    t02.y = real_two
    real_ne_1_2
    Real.1 != real_two
    t01.y != t02.y
    t01 != t02
    point2_ne_of_y_ne(t02, t03)
    t02.y = real_two
    t03.y = real_three
    real_ne_2_3
    real_two != real_three
    t02.y != t03.y
    t02 != t03
    point2_ne_of_y_ne(t01, t03)
    t01.y = Real.1
    t03.y = real_three
    real_ne_1_3
    Real.1 != real_three
    t01.y != t03.y
    t01 != t03
    t01 != t02 and t02 != t03 and t01 != t03
}

/// The left edge of the triangle has three lattice points.
theorem triangle_left_card {
    fs_card(triangle_left) = Nat.3
} by {
    fs_card_empty[Point2[Real]]
    fs_card(FiniteSet.empty[Point2[Real]]) = Nat.0
    fs_card_cardinality_is(FiniteSet.empty[Point2[Real]])
    FiniteSet.empty[Point2[Real]].cardinality_is(fs_card(FiniteSet.empty[Point2[Real]]))
    FiniteSet.empty[Point2[Real]].cardinality_is(Nat.0)
    finite_set_empty_contains_eq(t03)
    not FiniteSet.empty[Point2[Real]].contains(t03)
    finite_set_insert_cardinality_is_suc_of_not_contains(FiniteSet.empty[Point2[Real]], t03, Nat.0)
    fs_insert(FiniteSet.empty[Point2[Real]], t03).cardinality_is(Nat.1)
    triangle_left_distinct
    t02 != t03
    fs_insert_contains_eq(FiniteSet.empty[Point2[Real]], t03, t02)
    fs_insert(FiniteSet.empty[Point2[Real]], t03).contains(t02) =
        (t02 = t03 or FiniteSet.empty[Point2[Real]].contains(t02))
    finite_set_empty_contains_eq(t02)
    FiniteSet.empty[Point2[Real]].contains(t02) = false
    not fs_insert(FiniteSet.empty[Point2[Real]], t03).contains(t02)
    finite_set_insert_cardinality_is_suc_of_not_contains(
        fs_insert(FiniteSet.empty[Point2[Real]], t03), t02, Nat.1)
    fs_insert(fs_insert(FiniteSet.empty[Point2[Real]], t03), t02).cardinality_is(Nat.2)
    t01 != t03
    t01 != t02
    fs_insert_contains_eq(fs_insert(FiniteSet.empty[Point2[Real]], t03), t02, t01)
    fs_insert(fs_insert(FiniteSet.empty[Point2[Real]], t03), t02).contains(t01) =
        (t01 = t02 or fs_insert(FiniteSet.empty[Point2[Real]], t03).contains(t01))
    fs_insert_contains_eq(FiniteSet.empty[Point2[Real]], t03, t01)
    fs_insert(FiniteSet.empty[Point2[Real]], t03).contains(t01) =
        (t01 = t03 or FiniteSet.empty[Point2[Real]].contains(t01))
    finite_set_empty_contains_eq(t01)
    FiniteSet.empty[Point2[Real]].contains(t01) = false
    not fs_insert(fs_insert(FiniteSet.empty[Point2[Real]], t03), t02).contains(t01)
    finite_set_insert_cardinality_is_suc_of_not_contains(
        fs_insert(fs_insert(FiniteSet.empty[Point2[Real]], t03), t02), t01, Nat.2)
    fs_insert(fs_insert(fs_insert(FiniteSet.empty[Point2[Real]], t03), t02), t01).cardinality_is(Nat.2 + Nat.1)
    Nat.2 + Nat.1 = Nat.3
    fs_insert(fs_insert(fs_insert(FiniteSet.empty[Point2[Real]], t03), t02), t01).cardinality_is(Nat.3)
    triangle_left = fs_insert(fs_insert(fs_insert(
        FiniteSet.empty[Point2[Real]], t03), t02), t01)
    triangle_left.cardinality_is(Nat.3)
    fs_card_eq_of_cardinality_is(triangle_left, Nat.3)
    fs_card(triangle_left) = Nat.3
}

/// The left edge of the triangle contains exactly its three grid points.
theorem triangle_left_contains_eq(x: Point2[Real]) {
    triangle_left.contains(x) =
        (x = t01 or (x = t02 or x = t03))
} by {
    fs_insert_contains_eq(fs_insert(fs_insert(
        FiniteSet.empty[Point2[Real]], t03), t02), t01, x)
    triangle_left.contains(x) =
        (x = t01 or fs_insert(fs_insert(
            FiniteSet.empty[Point2[Real]], t03), t02).contains(x))
    fs_insert_contains_eq(fs_insert(FiniteSet.empty[Point2[Real]], t03), t02, x)
    fs_insert(fs_insert(FiniteSet.empty[Point2[Real]], t03), t02).contains(x) =
        (x = t02 or fs_insert(FiniteSet.empty[Point2[Real]], t03).contains(x))
    fs_insert_contains_eq(FiniteSet.empty[Point2[Real]], t03, x)
    fs_insert(FiniteSet.empty[Point2[Real]], t03).contains(x) =
        (x = t03 or FiniteSet.empty[Point2[Real]].contains(x))
    finite_set_empty_contains_eq(x)
    FiniteSet.empty[Point2[Real]].contains(x) = false
    triangle_left.contains(x) =
        (x = t01 or (x = t02 or (x = t03 or false)))
    or_false(x = t03)
    (x = t03 or false) = (x = t03)
    triangle_left.contains(x) =
        (x = t01 or (x = t02 or x = t03))
}

/// Every point of the left edge has a nonzero y-coordinate.
theorem triangle_left_member_y_ne_zero(x: Point2[Real]) {
    triangle_left.contains(x) implies x.y != Real.0
} by {
    if triangle_left.contains(x) {
        triangle_left_contains_eq(x)
        triangle_left.contains(x) =
            (x = t01 or (x = t02 or x = t03))
        x = t01 or (x = t02 or x = t03)
        if x = t01 {
            x = t01
            t01.y = Real.1
            real_ne_one_zero
            Real.1 != Real.0
            x.y != Real.0
        } else {
            if x = t02 {
                x = t02
                t02.y = real_two
                real_ne_2_0
                real_two != Real.0
                x.y != Real.0
            } else {
                x = t03
                t03.y = real_three
                real_ne_3_0
                real_three != Real.0
                x.y != Real.0
            }
        }
    }
}

/// The bottom and left edges of the triangle are disjoint.
theorem triangle_edges_disjoint {
    triangle_bottom.is_disjoint(triangle_left)
} by {
    triangle_bottom.is_disjoint(triangle_left) =
        triangle_bottom.underlying_set.is_disjoint(triangle_left.underlying_set)
    triangle_bottom.underlying_set.is_disjoint(triangle_left.underlying_set) = forall(x: Point2[Real]) {
        not (triangle_bottom.underlying_set.contains(x) and triangle_left.underlying_set.contains(x))
    }
    forall(x: Point2[Real]) {
        triangle_bottom.contains(x) = triangle_bottom.underlying_set.contains(x)
        triangle_left.contains(x) = triangle_left.underlying_set.contains(x)
        if triangle_bottom.underlying_set.contains(x) and triangle_left.underlying_set.contains(x) {
            triangle_bottom.contains(x)
            triangle_bottom_member_y_zero(x)
            x.y = Real.0
            triangle_left.contains(x)
            triangle_left_member_y_ne_zero(x)
            x.y != Real.0
            false
        }
        not (triangle_bottom.underlying_set.contains(x) and triangle_left.underlying_set.contains(x))
    }
    triangle_bottom.underlying_set.is_disjoint(triangle_left.underlying_set)
    triangle_bottom.is_disjoint(triangle_left)
}

/// The triangle has exactly eight boundary lattice points.
theorem triangle_boundary_card {
    fs_card(triangle_boundary) = Nat.8
} by {
    triangle_bottom_card
    fs_card(triangle_bottom) = Nat.5
    fs_card_cardinality_is(triangle_bottom)
    triangle_bottom.cardinality_is(fs_card(triangle_bottom))
    triangle_bottom.cardinality_is(Nat.5)
    triangle_left_card
    fs_card(triangle_left) = Nat.3
    fs_card_cardinality_is(triangle_left)
    triangle_left.cardinality_is(fs_card(triangle_left))
    triangle_left.cardinality_is(Nat.3)
    triangle_edges_disjoint
    triangle_bottom.is_disjoint(triangle_left)
    finite_set_disjoint_union_cardinality_is(triangle_bottom, triangle_left, Nat.5, Nat.3)
    triangle_bottom.cardinality_is(Nat.5) and triangle_left.cardinality_is(Nat.3) and
        triangle_bottom.is_disjoint(triangle_left)
    fs_union(triangle_bottom, triangle_left).cardinality_is(Nat.5 + Nat.3)
    Nat.5 + Nat.3 = Nat.8
    fs_union(triangle_bottom, triangle_left).cardinality_is(Nat.8)
    triangle_boundary = fs_union(triangle_bottom, triangle_left)
    triangle_boundary.cardinality_is(Nat.8)
    fs_card_eq_of_cardinality_is(triangle_boundary, Nat.8)
    fs_card(triangle_boundary) = Nat.8
}

/// The interior points of the triangle are distinct.
theorem triangle_interior_distinct {
    p11 != p12 and p12 != p21 and p11 != p21
} by {
    point2_ne_of_y_ne(p11, p12)
    p11.y = Real.1
    p12.y = real_two
    real_ne_1_2
    Real.1 != real_two
    p11.y != p12.y
    p11 != p12
    point2_ne_of_x_ne(p12, p21)
    p12.x = Real.1
    p21.x = real_two
    real_ne_1_2
    Real.1 != real_two
    p12.x != p21.x
    p12 != p21
    point2_ne_of_x_ne(p11, p21)
    p11.x = Real.1
    p21.x = real_two
    real_ne_1_2
    Real.1 != real_two
    p11.x != p21.x
    p11 != p21
    p11 != p12 and p12 != p21 and p11 != p21
}

/// The triangle has exactly three interior lattice points.
theorem triangle_interior_card {
    fs_card(triangle_interior) = Nat.3
} by {
    fs_card_empty[Point2[Real]]
    fs_card(FiniteSet.empty[Point2[Real]]) = Nat.0
    fs_card_cardinality_is(FiniteSet.empty[Point2[Real]])
    FiniteSet.empty[Point2[Real]].cardinality_is(fs_card(FiniteSet.empty[Point2[Real]]))
    FiniteSet.empty[Point2[Real]].cardinality_is(Nat.0)
    finite_set_empty_contains_eq(p21)
    not FiniteSet.empty[Point2[Real]].contains(p21)
    finite_set_insert_cardinality_is_suc_of_not_contains(FiniteSet.empty[Point2[Real]], p21, Nat.0)
    fs_insert(FiniteSet.empty[Point2[Real]], p21).cardinality_is(Nat.1)
    triangle_interior_distinct
    p12 != p21
    fs_insert_contains_eq(FiniteSet.empty[Point2[Real]], p21, p12)
    fs_insert(FiniteSet.empty[Point2[Real]], p21).contains(p12) =
        (p12 = p21 or FiniteSet.empty[Point2[Real]].contains(p12))
    finite_set_empty_contains_eq(p12)
    FiniteSet.empty[Point2[Real]].contains(p12) = false
    not fs_insert(FiniteSet.empty[Point2[Real]], p21).contains(p12)
    finite_set_insert_cardinality_is_suc_of_not_contains(
        fs_insert(FiniteSet.empty[Point2[Real]], p21), p12, Nat.1)
    fs_insert(fs_insert(FiniteSet.empty[Point2[Real]], p21), p12).cardinality_is(Nat.2)
    p11 != p21
    p11 != p12
    fs_insert_contains_eq(fs_insert(FiniteSet.empty[Point2[Real]], p21), p12, p11)
    fs_insert(fs_insert(FiniteSet.empty[Point2[Real]], p21), p12).contains(p11) =
        (p11 = p12 or fs_insert(FiniteSet.empty[Point2[Real]], p21).contains(p11))
    fs_insert_contains_eq(FiniteSet.empty[Point2[Real]], p21, p11)
    fs_insert(FiniteSet.empty[Point2[Real]], p21).contains(p11) =
        (p11 = p21 or FiniteSet.empty[Point2[Real]].contains(p11))
    finite_set_empty_contains_eq(p11)
    FiniteSet.empty[Point2[Real]].contains(p11) = false
    not fs_insert(fs_insert(FiniteSet.empty[Point2[Real]], p21), p12).contains(p11)
    finite_set_insert_cardinality_is_suc_of_not_contains(
        fs_insert(fs_insert(FiniteSet.empty[Point2[Real]], p21), p12), p11, Nat.2)
    fs_insert(fs_insert(fs_insert(FiniteSet.empty[Point2[Real]], p21), p12), p11).cardinality_is(Nat.2 + Nat.1)
    Nat.2 + Nat.1 = Nat.3
    fs_insert(fs_insert(fs_insert(FiniteSet.empty[Point2[Real]], p21), p12), p11).cardinality_is(Nat.3)
    triangle_interior = fs_insert(fs_insert(fs_insert(
        FiniteSet.empty[Point2[Real]], p21), p12), p11)
    triangle_interior.cardinality_is(Nat.3)
    fs_card_eq_of_cardinality_is(triangle_interior, Nat.3)
    fs_card(triangle_interior) = Nat.3
}

/// The interior of the triangle contains exactly its three grid points.
theorem triangle_interior_contains_eq(x: Point2[Real]) {
    triangle_interior.contains(x) =
        (x = p11 or (x = p12 or x = p21))
} by {
    fs_insert_contains_eq(fs_insert(fs_insert(
        FiniteSet.empty[Point2[Real]], p21), p12), p11, x)
    triangle_interior.contains(x) =
        (x = p11 or fs_insert(fs_insert(
            FiniteSet.empty[Point2[Real]], p21), p12).contains(x))
    fs_insert_contains_eq(fs_insert(FiniteSet.empty[Point2[Real]], p21), p12, x)
    fs_insert(fs_insert(FiniteSet.empty[Point2[Real]], p21), p12).contains(x) =
        (x = p12 or fs_insert(FiniteSet.empty[Point2[Real]], p21).contains(x))
    fs_insert_contains_eq(FiniteSet.empty[Point2[Real]], p21, x)
    fs_insert(FiniteSet.empty[Point2[Real]], p21).contains(x) =
        (x = p21 or FiniteSet.empty[Point2[Real]].contains(x))
    finite_set_empty_contains_eq(x)
    FiniteSet.empty[Point2[Real]].contains(x) = false
    triangle_interior.contains(x) =
        (x = p11 or (x = p12 or (x = p21 or false)))
    or_false(x = p21)
    (x = p21 or false) = (x = p21)
    triangle_interior.contains(x) =
        (x = p11 or (x = p12 or x = p21))
}

/// Fourteen minus two is twelve.
theorem real_fourteen_minus_two_is_twelve {
    real_fourteen - real_two = real_twelve
} by {
    real_fourteen = real_twelve + real_two
    real_fourteen - real_two = (real_twelve + real_two) - real_two
    (real_twelve + real_two) - real_two = real_twelve
    real_fourteen - real_two = real_twelve
}

/// The triangle has eight boundary lattice points.
theorem triangle_boundary_lattice_count {
    lattice_point_count(triangle_boundary) = Nat.8
} by {
    lattice_point_count_of_lattice_set(triangle_boundary)
    forall(p: Point2[Real]) {
        if triangle_boundary.contains(p) {
            finite_set_union_contains_eq(triangle_bottom, triangle_left, p)
            triangle_boundary.contains(p) =
                (triangle_bottom.contains(p) or triangle_left.contains(p))
            triangle_bottom.contains(p) or triangle_left.contains(p)
            if triangle_bottom.contains(p) {
                triangle_bottom_contains_eq(p)
                triangle_bottom.contains(p) =
                    (p = t00 or (p = t10 or (p = t20 or (p = t30 or p = t40))))
                p = t00 or (p = t10 or (p = t20 or (p = t30 or p = t40)))
                if p = t00 {
                    is_lattice_point_t00
                    is_lattice_point(p)
                } else {
                    if p = t10 {
                        is_lattice_point_t10
                        is_lattice_point(p)
                    } else {
                        if p = t20 {
                            is_lattice_point_t20
                            is_lattice_point(p)
                        } else {
                            if p = t30 {
                                is_lattice_point_t30
                                is_lattice_point(p)
                            } else {
                                p = t40
                                is_lattice_point_t40
                                is_lattice_point(p)
                            }
                        }
                    }
                }
            } else {
                triangle_left_contains_eq(p)
                triangle_left.contains(p) =
                    (p = t01 or (p = t02 or p = t03))
                p = t01 or (p = t02 or p = t03)
                if p = t01 {
                    is_lattice_point_t01
                    is_lattice_point(p)
                } else {
                    if p = t02 {
                        is_lattice_point_t02
                        is_lattice_point(p)
                    } else {
                        p = t03
                        is_lattice_point_t03
                        is_lattice_point(p)
                    }
                }
            }
            is_lattice_point(p)
        }
    }
    lattice_point_count(triangle_boundary) = fs_card(triangle_boundary)
    triangle_boundary_card
    lattice_point_count(triangle_boundary) = Nat.8
}

/// The triangle has three interior lattice points.
theorem triangle_interior_lattice_count {
    lattice_point_count(triangle_interior) = Nat.3
} by {
    lattice_point_count_of_lattice_set(triangle_interior)
    forall(p: Point2[Real]) {
        if triangle_interior.contains(p) {
            triangle_interior_contains_eq(p)
            triangle_interior.contains(p) =
                (p = p11 or (p = p12 or p = p21))
            p = p11 or (p = p12 or p = p21)
            if p = p11 {
                is_lattice_point_p11
                is_lattice_point(p)
            } else {
                if p = p12 {
                    is_lattice_point_p12
                    is_lattice_point(p)
                } else {
                    p = p21
                    is_lattice_point_p21
                    is_lattice_point(p)
                }
            }
        }
    }
    lattice_point_count(triangle_interior) = fs_card(triangle_interior)
    triangle_interior_card
    lattice_point_count(triangle_interior) = Nat.3
}

/// Pick's theorem for the 3-4-5 triangle:
/// `Area = 6 = I + B/2 - 1 = 3 + 8/2 - 1`.
theorem pick_right_triangle {
    is_simple_lattice_polygon(right_triangle) implies
    polygon_area(right_triangle) =
        from_nat[Real](lattice_point_count(triangle_interior)) +
        from_nat[Real](lattice_point_count(triangle_boundary)) / real_two - Real.1
} by {
    if is_simple_lattice_polygon(right_triangle) {
        right_triangle_area_is_six
        polygon_area(right_triangle) = real_six
        triangle_interior_lattice_count
        lattice_point_count(triangle_interior) = Nat.3
        triangle_boundary_lattice_count
        lattice_point_count(triangle_boundary) = Nat.8
        from_nat_three
        from_nat[Real](Nat.3) = real_three
        from_nat[Real](lattice_point_count(triangle_interior)) = real_three
        from_nat_eight
        from_nat[Real](Nat.8) = real_eight
        from_nat[Real](lattice_point_count(triangle_boundary)) = real_eight
        from_nat[Real](lattice_point_count(triangle_boundary)) / real_two = real_eight / real_two
        real_eight_div_two
        real_eight / real_two = real_four
        from_nat[Real](lattice_point_count(triangle_boundary)) / real_two = real_four
        from_nat[Real](lattice_point_count(triangle_interior)) +
            from_nat[Real](lattice_point_count(triangle_boundary)) / real_two - Real.1 =
            real_three + real_four - Real.1
        real_four = real_three + Real.1
        real_four - Real.1 = (real_three + Real.1) - Real.1
        (real_three + Real.1) - Real.1 = real_three
        real_four - Real.1 = real_three
        real_three + real_four - Real.1 = real_three + real_three
        real_three_plus_three_is_six
        real_three + real_three = real_six
        real_three + real_four - Real.1 = real_six
        polygon_area(right_triangle) =
            from_nat[Real](lattice_point_count(triangle_interior)) +
            from_nat[Real](lattice_point_count(triangle_boundary)) / real_two - Real.1
    }
}

/// Pick's theorem for the 3-4-5 triangle in the doubled-area form:
/// `2 * Area = 2I + B - 2 = 6 + 8 - 2`.
theorem pick_right_triangle_doubled {
    is_simple_lattice_polygon(right_triangle) implies
    point2_polygon_area2(right_triangle) =
        from_nat[Real](lattice_point_count(triangle_interior) +
            lattice_point_count(triangle_interior) +
            lattice_point_count(triangle_boundary)) - real_two
} by {
    if is_simple_lattice_polygon(right_triangle) {
        right_triangle_area2_is_twelve
        point2_polygon_area2(right_triangle) = real_twelve
        triangle_interior_lattice_count
        lattice_point_count(triangle_interior) = Nat.3
        triangle_boundary_lattice_count
        lattice_point_count(triangle_boundary) = Nat.8
        from_nat_add[Real](lattice_point_count(triangle_interior) + lattice_point_count(triangle_interior),
            lattice_point_count(triangle_boundary))
        from_nat[Real](lattice_point_count(triangle_interior) +
            lattice_point_count(triangle_interior) +
            lattice_point_count(triangle_boundary)) =
            from_nat[Real](lattice_point_count(triangle_interior) + lattice_point_count(triangle_interior)) +
            from_nat[Real](lattice_point_count(triangle_boundary))
        from_nat_add[Real](lattice_point_count(triangle_interior), lattice_point_count(triangle_interior))
        from_nat[Real](lattice_point_count(triangle_interior) + lattice_point_count(triangle_interior)) =
            from_nat[Real](lattice_point_count(triangle_interior)) + from_nat[Real](lattice_point_count(triangle_interior))
        from_nat_three
        from_nat[Real](Nat.3) = real_three
        from_nat[Real](lattice_point_count(triangle_interior)) = real_three
        from_nat[Real](lattice_point_count(triangle_interior)) + from_nat[Real](lattice_point_count(triangle_interior)) =
            real_three + real_three
        real_three_plus_three_is_six
        real_three + real_three = real_six
        from_nat[Real](lattice_point_count(triangle_interior) + lattice_point_count(triangle_interior)) = real_six
        from_nat_eight
        from_nat[Real](Nat.8) = real_eight
        from_nat[Real](lattice_point_count(triangle_boundary)) = real_eight
        from_nat[Real](lattice_point_count(triangle_interior) + lattice_point_count(triangle_interior)) +
            from_nat[Real](lattice_point_count(triangle_boundary)) = real_six + real_eight
        real_six_plus_eight_is_fourteen
        real_six + real_eight = real_fourteen
        from_nat[Real](lattice_point_count(triangle_interior) +
            lattice_point_count(triangle_interior) +
            lattice_point_count(triangle_boundary)) = real_fourteen
        from_nat[Real](lattice_point_count(triangle_interior) +
            lattice_point_count(triangle_interior) +
            lattice_point_count(triangle_boundary)) - real_two = real_fourteen - real_two
        real_fourteen_minus_two_is_twelve
        real_fourteen - real_two = real_twelve
        point2_polygon_area2(right_triangle) =
            from_nat[Real](lattice_point_count(triangle_interior) +
                lattice_point_count(triangle_interior) +
                lattice_point_count(triangle_boundary)) - real_two
    }
}

// ---------------------------------------------------------------------------
// Pick's theorem, general form (statement recorded, proof not yet possible).
// ---------------------------------------------------------------------------
//
// The library cannot yet express "the finite set of lattice points inside an
// arbitrary simple polygon": that needs a general simplicity predicate for
// polygons, a boundedness argument (every simple polygon lies in the bounding
// box of its vertices), and the finiteness of the lattice points inside a
// bounded region.  With such machinery, `interior_lattice_count(points)` and
// `boundary_lattice_count(points)` would be `lattice_point_count` of the finite
// sets of lattice points strictly inside (the region minus the boundary, see
// `point_in_polygon_region` and `point_on_polygon_boundary`) and on the
// boundary of the polygon, and the theorem would read:
//
//     theorem pick_theorem(points: List[Point2[Real]]) {
//         is_simple_lattice_polygon(points) implies
//         polygon_area(points) =
//             from_nat[Real](interior_lattice_count(points)) +
//             from_nat[Real](boundary_lattice_count(points)) / real_two - Real.1
//     }
//
// Multiplying by two gives the equivalent doubled-area form, which is the form
// native to the library's shoelace area `point2_polygon_area2` and the form in
// which the small cases above are verified:
//
//     theorem pick_theorem_doubled(points: List[Point2[Real]]) {
//         is_simple_lattice_polygon(points) implies
//         point2_polygon_area2(points) =
//             from_nat[Real](interior_lattice_count(points) +
//                 interior_lattice_count(points) +
//                 boundary_lattice_count(points)) - real_two
//     }
//
// The three instances verified above (the unit square, the 1-by-2 rectangle,
// and the 3-4-5 right triangle) instantiate exactly this pattern, with
// `interior_lattice_count` and `boundary_lattice_count` taken as
// `lattice_point_count` of the explicit finite sets
// `unit_square_interior`/`unit_square_boundary`,
// `unit_rect_interior`/`unit_rect_boundary`, and
// `triangle_interior`/`triangle_boundary`.
