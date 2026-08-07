/// Basic convex-hull closure and minimality facts for Point2 point sets.

from ordered_field import OrderedField
from geometry.point2 import Point2
from geometry.point2_convex_translate_bridge import point2_convex_hull,
    point2_convex_hull_contains, point2_convex_hull_contains_eq,
    point2_point_set_convex
from data.basic.set import Set, subset_antisymm

/// Every point of a set lies in its convex hull.
theorem point2_convex_hull_contains_of_contains[T: OrderedField](s: Set[Point2[T]], p: Point2[T]) {
    s.contains(p) implies point2_convex_hull(s).contains(p)
} by {
    if s.contains(p) {
        forall(c: Set[Point2[T]]) {
            if point2_point_set_convex(c) and s.subset(c) {
                s.subset(c)
                c.contains(p)
            }
        }
        point2_convex_hull_contains(s, p)
        point2_convex_hull_contains_eq(s, p)
        point2_convex_hull(s).contains(p)
    }
}

/// The convex hull is contained in every convex superset.
theorem point2_convex_hull_subset_of_convex_superset[T: OrderedField](s: Set[Point2[T]], c: Set[Point2[T]]) {
    point2_point_set_convex(c) and s.subset(c) implies point2_convex_hull(s).subset(c)
} by {
    if point2_point_set_convex(c) and s.subset(c) {
        forall(p: Point2[T]) {
            if point2_convex_hull(s).contains(p) {
                point2_convex_hull_contains_eq(s, p)
                point2_convex_hull_contains(s, p)
                point2_convex_hull_contains(s, p) = forall(d: Set[Point2[T]]) {
                    point2_point_set_convex(d) and s.subset(d) implies d.contains(p)
                }
                forall(d: Set[Point2[T]]) {
                    point2_point_set_convex(d) and s.subset(d) implies d.contains(p)
                }
                c.contains(p)
            }
        }
    }
}

/// The convex hull is convex.
theorem point2_convex_hull_convex[T: OrderedField](s: Set[Point2[T]]) {
    point2_point_set_convex(point2_convex_hull(s))
} by {
    forall(a: Point2[T], b: Point2[T], t: T) {
        if point2_convex_hull(s).contains(a) and point2_convex_hull(s).contains(b) and T.0 <= t and t <= T.1 {
            forall(c: Set[Point2[T]]) {
                if point2_point_set_convex(c) and s.subset(c) {
                    point2_convex_hull_contains_eq(s, a)
                    point2_convex_hull_contains(s, a)
                    point2_convex_hull_contains(s, a) = forall(d: Set[Point2[T]]) {
                        point2_point_set_convex(d) and s.subset(d) implies d.contains(a)
                    }
                    forall(d: Set[Point2[T]]) {
                        point2_point_set_convex(d) and s.subset(d) implies d.contains(a)
                    }
                    c.contains(a)

                    point2_convex_hull_contains_eq(s, b)
                    point2_convex_hull_contains(s, b)
                    point2_convex_hull_contains(s, b) = forall(d: Set[Point2[T]]) {
                        point2_point_set_convex(d) and s.subset(d) implies d.contains(b)
                    }
                    forall(d: Set[Point2[T]]) {
                        point2_point_set_convex(d) and s.subset(d) implies d.contains(b)
                    }
                    c.contains(b)

                    point2_point_set_convex(c) = forall(x: Point2[T], y: Point2[T], u: T) {
                        c.contains(x) and c.contains(y) and T.0 <= u and u <= T.1 implies
                        c.contains(x.param_line(y, u))
                    }
                    c.contains(a.param_line(b, t))
                }
            }
            point2_convex_hull_contains(s, a.param_line(b, t))
            point2_convex_hull_contains_eq(s, a.param_line(b, t))
            point2_convex_hull(s).contains(a.param_line(b, t))
        }
    }
}

/// Convex hull is monotone with respect to set inclusion.
theorem point2_convex_hull_monotone[T: OrderedField](s: Set[Point2[T]], t: Set[Point2[T]]) {
    s.subset(t) implies point2_convex_hull(s).subset(point2_convex_hull(t))
} by {
    if s.subset(t) {
        point2_convex_hull_convex(t)
        forall(p: Point2[T]) {
            if s.contains(p) {
                t.contains(p)
                point2_convex_hull_contains_of_contains(t, p)
                point2_convex_hull(t).contains(p)
            }
        }
        s.subset(point2_convex_hull(t))
        point2_convex_hull_subset_of_convex_superset(s, point2_convex_hull(t))
        point2_convex_hull(s).subset(point2_convex_hull(t))
    }
}

/// Taking the convex hull twice gives the same set.
theorem point2_convex_hull_idempotent[T: OrderedField](s: Set[Point2[T]]) {
    point2_convex_hull(point2_convex_hull(s)) = point2_convex_hull(s)
} by {
    point2_convex_hull_convex(s)
    forall(p: Point2[T]) {
        if point2_convex_hull(s).contains(p) {
            point2_convex_hull_contains_of_contains(point2_convex_hull(s), p)
            point2_convex_hull(point2_convex_hull(s)).contains(p)
        }
    }
    point2_convex_hull(s).subset(point2_convex_hull(point2_convex_hull(s)))

    forall(p: Point2[T]) {
        if point2_convex_hull(s).contains(p) {
            point2_convex_hull(s).contains(p)
        }
    }
    point2_convex_hull(s).subset(point2_convex_hull(s))
    point2_convex_hull_subset_of_convex_superset(point2_convex_hull(s), point2_convex_hull(s))
    point2_convex_hull(point2_convex_hull(s)).subset(point2_convex_hull(s))

    subset_antisymm(point2_convex_hull(point2_convex_hull(s)), point2_convex_hull(s))
}

/// A convex set is equal to its convex hull.
theorem point2_convex_hull_eq_self_of_convex[T: OrderedField](s: Set[Point2[T]]) {
    point2_point_set_convex(s) implies point2_convex_hull(s) = s
} by {
    if point2_point_set_convex(s) {
        forall(p: Point2[T]) {
            if s.contains(p) {
                point2_convex_hull_contains_of_contains(s, p)
                point2_convex_hull(s).contains(p)
            }
        }
        s.subset(point2_convex_hull(s))

        forall(p: Point2[T]) {
            if s.contains(p) {
                s.contains(p)
            }
        }
        s.subset(s)
        point2_point_set_convex(s) and s.subset(s)
        point2_convex_hull_subset_of_convex_superset(s, s)
        point2_convex_hull(s).subset(s)

        point2_convex_hull(s).subset(s) and s.subset(point2_convex_hull(s))
        subset_antisymm(point2_convex_hull(s), s)
        point2_convex_hull(s) = s
    }
}
