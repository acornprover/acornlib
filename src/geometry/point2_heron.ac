from comm_ring import CommRing
from algebra.add_comm_group import AddCommGroup
from ordered_field import OrderedField
from geometry.point2 import Point2, point2_ext
from geometry.point2_algebra import point2_dot_comm,
    point2_norm_sq_sub_expansion, point2_ring_square_add
from geometry.point2_triangle import point2_triangle_area2_eq_orientation

/// Removing a common scalar base from two differences gives the endpoint difference.
theorem point2_scalar_sub_same_base[T: AddCommGroup](a: T, b: T, c: T) {
    (c - a) - (b - a) = c - b
} by {
    (c - a) - (b - a) = (c + -a) + (a + -b)
    (c + -a) + (a + -b) = c + (-a + a) + -b
    -a + a = T.0
}

/// Removing a common base from two displacements gives the displacement between endpoints.
theorem point2_sub_same_base[T: AddCommGroup](a: Point2[T], b: Point2[T], c: Point2[T]) {
    c.sub(a).sub(b.sub(a)) = c.sub(b)
} by {
    let lhs = c.sub(a).sub(b.sub(a))
    let rhs = c.sub(b)
    lhs.x = (c.x - a.x) - (b.x - a.x)
    rhs.x = c.x - b.x
    point2_scalar_sub_same_base(a.x, b.x, c.x)
    (c.x - a.x) - (b.x - a.x) = c.x - b.x
    lhs.x = rhs.x
    lhs.y = (c.y - a.y) - (b.y - a.y)
    rhs.y = c.y - b.y
    point2_scalar_sub_same_base(a.y, b.y, c.y)
    (c.y - a.y) - (b.y - a.y) = c.y - b.y
    lhs.y = rhs.y
    point2_ext(lhs, rhs)
}

/// A subtracted middle term cancels with the same term added back.
theorem point2_cancel_sub_add[T: AddCommGroup](x: T, m: T, z: T) {
    (x - m) + (m + z) = x + z
} by {
    x + ((-m + m) + z) = x + (T.0 + z)
}

/// Subtracting after an addition can move the second summand first.
theorem point2_add_sub_rearrange[T: AddCommGroup](a: T, b: T, c: T) {
    (a + b) - c = b + (a - c)
} by {
    a + b + -c = b + (a + -c)
}

/// A difference from a difference can be rearranged around the removed middle term.
theorem point2_sub_sub_rearrange[T: AddCommGroup](x: T, y: T, z: T) {
    x - (y - z) = z - (y - x)
} by {
    x - (y - z) = x + -(y + -z)
    -(y + -z) = z + -y
    x + (z + -y) = z + (x + -y)
    -(y + -x) = x + -y
}

/// Subtracting two terms successively is subtracting their sum.
theorem point2_sub_sub_eq_sub_add[T: AddCommGroup](x: T, y: T, z: T) {
    x - y - z = x - (y + z)
} by {
    x - y - z = x + -y + -z
    -z + -y = -y + -z
}

/// Adding and then subtracting the same term on the right leaves the base term.
theorem point2_add_sub_right_cancel[T: AddCommGroup](x: T, d: T) {
    x + d - d = x
} by {
}

/// If one term is a difference from a sum, subtracting it recovers the removed term.
theorem point2_sub_eq_sub_imp_sub_eq[T: AddCommGroup](sum: T, removed: T, x: T) {
    x = sum - removed implies sum - x = removed
} by {
    if x = sum - removed {
        point2_sub_sub_rearrange(sum, sum, removed)
        sum - sum = T.0
        sum - x = removed
    }
}

/// Four copies of a sum minus four copies of one summand leaves four copies of the other.
theorem point2_four_sum_sub_four_from_sum[T: AddCommGroup](a: T, d: T, p: T) {
    a + d = p implies p + p + (p + p) - (d + d + (d + d)) = a + a + (a + a)
} by {
    if a + d = p {
        point2_add_sub_right_cancel(a + a + (a + a), d + d + (d + d))
        a + a + (a + a) + (d + d + (d + d)) - (d + d + (d + d)) = a + a + (a + a)
    }
}

/// Opposite middle terms cancel in a four-term Lagrange expansion.
theorem point2_lagrange_cancel_middle[T: AddCommGroup](x: T, y: T, z: T, w: T, m: T, n: T) {
    (x - m - n + y) + (z + m + n + w) = x + y + z + w
} by {
    x - m = x + -m
    x - m - n = x + -m + -n
    x - m - n + y = (x - m) + (y - n)
    point2_cancel_sub_add(x, m, z)
    point2_cancel_sub_add(y, n, w)
    ((x - m) + (m + z)) + ((y - n) + (n + w)) = (x + z) + (y + w)
}

/// The scalar form of the two-dimensional Lagrange identity.
theorem point2_lagrange_scalar_identity[T: CommRing](a: T, b: T, c: T, d: T) {
    (a * d - b * c) * (a * d - b * c) + (a * c + b * d) * (a * c + b * d) =
        (a * a + b * b) * (c * c + d * d)
} by {
    let ad = a * d
    let bc = b * c
    let ac = a * c
    let bd = b * d
    point2_ring_square_add(ad, -bc)
    (ad - bc) * (ad - bc) = ad * ad - ad * bc - bc * ad + bc * bc
    point2_ring_square_add(ac, bd)
    (ac + bd) * (ac + bd) = ac * ac + ac * bd + bd * ac + bd * bd
    ad * bc = (a * d) * (b * c)
    (a * d) * (b * c) = a * b * c * d
    ac * bd = (a * c) * (b * d)
    (a * c) * (b * d) = a * b * c * d
    ad * bc = ac * bd
    bc * ad = (b * c) * (a * d)
    (b * c) * (a * d) = a * b * c * d
    bd * ac = (b * d) * (a * c)
    (b * d) * (a * c) = a * b * c * d
    bc * ad = bd * ac
    -(ad * bc) + ac * bd = T.0
    -(bc * ad) + bd * ac = T.0
    ac * ac + ac * bd + bd * ac + bd * bd = ac * ac + ad * bc + bc * ad + bd * bd
    point2_lagrange_cancel_middle(ad * ad, bc * bc, ac * ac, bd * bd, ad * bc, bc * ad)
    (ad * ad - ad * bc - bc * ad + bc * bc) +
        (ac * ac + ac * bd + bd * ac + bd * bd) =
        ad * ad + bc * bc + ac * ac + bd * bd
    ad * ad = a * a * (d * d)
    bc * bc = b * b * (c * c)
    ac * ac = a * a * (c * c)
    bd * bd = b * b * (d * d)
    ad * ad + bc * bc + ac * ac + bd * bd =
        a * a * (c * c) + a * a * (d * d) + (b * b * (c * c) + b * b * (d * d))
}

/// The square of a difference, with the two cross terms written separately.
theorem point2_ring_square_sub[T: CommRing](x: T, y: T) {
    (x - y) * (x - y) = x * x - x * y - y * x + y * y
} by {
    point2_ring_square_add(x, -y)
}

/// The product of conjugate linear factors is a difference of squares.
theorem point2_ring_difference_of_squares[T: CommRing](x: T, y: T) {
    (x + y) * (x - y) = x * x - y * y
} by {
    x * x + x * y - (x * y + y * y) = x * x + (x * y - x * y) - y * y
    x * y - x * y = T.0
    x * x + y * x - (x * y + y * y) = x * x - y * y
}

/// The two-dimensional Lagrange identity for dot and cross products.
theorem point2_lagrange_identity[T: CommRing](u: Point2[T], v: Point2[T]) {
    u.cross(v) * u.cross(v) + u.dot(v) * u.dot(v) = u.norm_sq * v.norm_sq
} by {
    let ux = u.x
    let uy = u.y
    let vx = v.x
    let vy = v.y
    u.norm_sq = ux * ux + uy * uy
    v.norm_sq = vx * vx + vy * vy
    point2_lagrange_scalar_identity(ux, uy, vx, vy)
}

/// The product of the four Heron linear factors.
define heron_linear_product[T: CommRing](x: T, y: T, z: T) -> T {
    (x + y + z) * (-x + y + z) * (x - y + z) * (x + y - z)
}

/// The first pair of Heron linear factors is a difference of squares.
theorem heron_first_pair_difference[T: CommRing](x: T, y: T, z: T) {
    (x + y + z) * (-x + y + z) = (y + z) * (y + z) - x * x
} by {
    -x + y + z = (y + z) - x
    point2_ring_difference_of_squares(y + z, x)
}

/// The second pair of Heron linear factors is a difference of squares.
theorem heron_second_pair_difference[T: CommRing](x: T, y: T, z: T) {
    (x - y + z) * (x + y - z) = x * x - (y - z) * (y - z)
} by {
    x - y + z = x - (y - z)
    point2_ring_difference_of_squares(x, y - z)
}

/// Expansions of the two binomial squares in Heron's factorization.
theorem heron_sum_and_diff_square_expansions[T: CommRing](y: T, z: T) {
    (y + z) * (y + z) = y * y + z * z + (y * z + y * z) and
    (y - z) * (y - z) = y * y + z * z - (y * z + y * z)
} by {
    point2_ring_square_add(y, z)
    (y + z) * (y + z) = y * y + y * z + (y * z + z * z)
    y * y + y * z + (y * z + z * z) = y * y + z * z + (y * z + y * z)
    point2_ring_square_sub(y, z)
    (y - z) * (y - z) = y * y - y * z - z * y + z * z
    z * y = y * z
    y * y - y * z - y * z + z * z = z * z + (y * y - y * z - y * z)
    z * z + (y * y - y * z - y * z) = y * y + z * z - y * z - y * z
    y * y - y * z - y * z + z * z = y * y + z * z - y * z - y * z
    y * y + z * z - y * z - y * z = y * y + z * z + -(y * z) + -(y * z)
    -(y * z + y * z) = -(y * z) + -(y * z)
    y * y + z * z - (y * z + y * z) = y * y + z * z + -(y * z + y * z)
    y * y + z * z + -(y * z + y * z) = y * y + z * z + (-(y * z) + -(y * z))
    y * y + z * z + (-(y * z) + -(y * z)) = y * y + z * z + -(y * z) + -(y * z)
    y * y + z * z - y * z - y * z = y * y + z * z - (y * z + y * z)
    y * y - y * z - y * z + z * z = y * y + z * z - (y * z + y * z)
}

/// Heron's linear-factor product as a difference of two squares.
theorem heron_linear_product_difference_of_squares[T: CommRing](x: T, y: T, z: T) {
    heron_linear_product(x, y, z) =
        (y * z + y * z) * (y * z + y * z) -
        (y * y + z * z - x * x) * (y * y + z * z - x * x)
} by {
    let sum_sq = y * y + z * z
    let double_yz = y * z + y * z
    let power = sum_sq - x * x
    heron_first_pair_difference(x, y, z)
    (x + y + z) * (-x + y + z) = (y + z) * (y + z) - x * x
    heron_second_pair_difference(x, y, z)
    (x - y + z) * (x + y - z) = x * x - (y - z) * (y - z)
    heron_sum_and_diff_square_expansions(y, z)
    (y + z) * (y + z) = sum_sq + double_yz
    (y - z) * (y - z) = sum_sq - double_yz
    point2_add_sub_rearrange(sum_sq, double_yz, x * x)
    (y + z) * (y + z) - x * x = double_yz + power
    point2_sub_sub_rearrange(x * x, sum_sq, double_yz)
    x * x - (y - z) * (y - z) = double_yz - power
    heron_linear_product(x, y, z) = (double_yz + power) * (double_yz - power)
    point2_ring_difference_of_squares(double_yz, power)
}

/// Doubling converts a difference of squares into the doubled remaining square.
theorem point2_double_square_difference_from_sum[T: CommRing](area: T, dot: T, prod: T) {
    area * area + dot * dot = prod * prod implies
        (prod + prod) * (prod + prod) - (dot + dot) * (dot + dot) =
        (area + area) * (area + area)
} by {
    if area * area + dot * dot = prod * prod {
        point2_ring_square_add(prod, prod)
        point2_ring_square_add(dot, dot)
        point2_ring_square_add(area, area)
        point2_four_sum_sub_four_from_sum(area * area, dot * dot, prod * prod)
        prod * prod + prod * prod + (prod * prod + prod * prod) -
            (dot * dot + dot * dot + (dot * dot + dot * dot)) =
            area * area + area * area + (area * area + area * area)
    }
}

/// The scalar Heron identity from Lagrange's identity and the chord-square relation.
theorem heron_from_lagrange_scalar[T: CommRing](area: T, dot: T, x: T, y: T, z: T) {
    y * y + z * z - x * x = dot + dot and
    area * area + dot * dot = (y * z) * (y * z)
    implies heron_linear_product(x, y, z) = (area + area) * (area + area)
} by {
    if y * y + z * z - x * x = dot + dot and
        area * area + dot * dot = (y * z) * (y * z) {
        heron_linear_product_difference_of_squares(x, y, z)
        point2_double_square_difference_from_sum(area, dot, y * z)
        (y * z + y * z) * (y * z + y * z) - (dot + dot) * (dot + dot) =
            (area + area) * (area + area)
    }
}

/// Heron's squared identity for two coordinate vectors with length witnesses.
theorem point2_heron_vectors[T: CommRing](u: Point2[T], v: Point2[T], x: T, y: T, z: T) {
    x * x = v.sub(u).norm_sq and y * y = v.norm_sq and z * z = u.norm_sq implies
        heron_linear_product(x, y, z) = (u.cross(v) + u.cross(v)) * (u.cross(v) + u.cross(v))
} by {
    if x * x = v.sub(u).norm_sq and y * y = v.norm_sq and z * z = u.norm_sq {
        let area = u.cross(v)
        let dot = u.dot(v)
        let sum_sq = y * y + z * z
        point2_norm_sq_sub_expansion(v, u)
        v.sub(u).norm_sq = v.norm_sq + u.norm_sq - v.dot(u) - v.dot(u)
        point2_dot_comm(v, u)
        v.dot(u) = dot
        x * x = sum_sq - dot - dot
        point2_sub_sub_eq_sub_add(sum_sq, dot, dot)
        x * x = sum_sq - (dot + dot)
        point2_sub_eq_sub_imp_sub_eq(sum_sq, dot + dot, x * x)
        sum_sq - x * x = dot + dot
        y * y + z * z - x * x = dot + dot

        point2_lagrange_identity(u, v)
        area * area + dot * dot = u.norm_sq * v.norm_sq
        u.norm_sq * v.norm_sq = (z * z) * (y * y)
        (z * z) * (y * y) = (y * z) * (y * z)
        heron_from_lagrange_scalar(area, dot, x, y, z)
        heron_linear_product(x, y, z) = (area + area) * (area + area)
        heron_linear_product(x, y, z) = (u.cross(v) + u.cross(v)) * (u.cross(v) + u.cross(v))
    }
}

/// Heron's squared identity for a triangle in the coordinate plane.
theorem point2_heron_triangle_squared[T: OrderedField](a: Point2[T], b: Point2[T], c: Point2[T], x: T, y: T, z: T) {
    x * x = c.sub(b).norm_sq and y * y = c.sub(a).norm_sq and z * z = b.sub(a).norm_sq implies
        heron_linear_product(x, y, z) =
        (a.triangle_area2(b, c) + a.triangle_area2(b, c)) *
        (a.triangle_area2(b, c) + a.triangle_area2(b, c))
} by {
    if x * x = c.sub(b).norm_sq and y * y = c.sub(a).norm_sq and z * z = b.sub(a).norm_sq {
        let u = b.sub(a)
        let v = c.sub(a)
        point2_sub_same_base(a, b, c)
        y * y = v.norm_sq
        z * z = u.norm_sq
        point2_heron_vectors(u, v, x, y, z)
        heron_linear_product(x, y, z) = (u.cross(v) + u.cross(v)) * (u.cross(v) + u.cross(v))
        point2_triangle_area2_eq_orientation(a, b, c)
        a.triangle_area2(b, c) = u.cross(v)
        heron_linear_product(x, y, z) =
            (a.triangle_area2(b, c) + a.triangle_area2(b, c)) *
            (a.triangle_area2(b, c) + a.triangle_area2(b, c))
    }
}
