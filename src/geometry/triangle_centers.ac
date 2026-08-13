/// Triangle centers: the centroid, circumcenter, and orthocenter of a
/// triangle in the coordinate plane, together with the internal angle
/// bisector theorem.
///
/// Status:
///   - The centroid is the common point of the three medians, and it
///     trisects each median: with `2m = b + c` and `3g = a + b + c`, the
///     centroid `g` satisfies `g - a = 2 (m - g)` for the median from `a`.
///   - The circumcenter: a point equidistant from two vertices lies on the
///     perpendicular bisector of the side between them, so a circumcenter of
///     a triangle lies on the perpendicular bisector of each side.
///   - The orthocenter: with `h = a + b + c - 2o` for a circumcenter `o`,
///     the point `h` lies on all three altitudes.
///   - The angle bisector theorem: the internal angle bisector at `a` meets
///     `bc` at `d` with `|b - d|^2 / |d - c|^2 = |b - a|^2 / |c - a|^2`.

from real import Real, mul_left_cancel
from geometry.point2 import Point2, point2_zero, point2_ext,
    point2_add_assoc, point2_add_comm, point2_add_zero_right,
    point2_sub_self
from geometry.point2_algebra import point2_add_four_last_next_to_first,
    point2_sub_add_sub, point2_sub_through,
    point2_sub_reverse_neg, point2_sub_eq_add_neg, point2_sub_zero_right,
    point2_dot_comm, point2_dot_add_left, point2_dot_add_right,
    point2_dot_sub_left, point2_dot_sub_right, point2_dot_neg_right, point2_norm_sq_sub_expansion,
    point2_dist_sq_eq_norm_sq_sub, point2_dist_sq_comm, point2_norm_sq_neg,
    point2_norm_sq_smul,
    point2_cross_add_left, point2_cross_add_right, point2_cross_smul_left,
    point2_cross_smul_right, point2_cross_swap, point2_cross_self
from geometry.point2_affine import point2_add_sub_left_cancel, point2_smul_one_left
from geometry.point2_circle import point2_on_circle_eq_dist_sq,
    point2_center_dist_imp_on_circle

numerals Real

// ---------------------------------------------------------------------------
// Small point-algebra helpers used by the center theorems.
// ---------------------------------------------------------------------------

/// Two is a nonzero real.
theorem triangle_real_two_neq_zero {
    Real.1 + Real.1 != Real.0
} by {
    Real.0 < Real.1
    Real.0 + Real.0 < Real.1 + Real.1
}

/// A doubled real equality cancels: `2x = 0` implies `x = 0`.
theorem triangle_double_zero_cancel(x: Real) {
    (Real.1 + Real.1) * x = Real.0 implies x = Real.0
} by {
    if (Real.1 + Real.1) * x = Real.0 {
        Real.0 = (Real.1 + Real.1) * x
        triangle_real_two_neq_zero
        Real.1 + Real.1 != Real.0
        mul_left_cancel(Real.0, Real.1 + Real.1, x)
        Real.0 / (Real.1 + Real.1) = x
        Real.0 / (Real.1 + Real.1) = Real.0
        x = Real.0
    }
}

/// The middle term cancels in a difference of paired sums:
/// `(a + b) - (b + c) = a - c`.
theorem triangle_real_add_sub_cancel_middle(a: Real, b: Real, c: Real) {
    (a + b) - (b + c) = a - c
} by {
    (a + b) - (b + c) = a + b + -(b + c)
    -(b + c) = -b + -c
    a + b + -(b + c) = a + b + (-b + -c)
    a + b + (-b + -c) = a + b + -b + -c
    point2_add_four_last_next_to_first(a, b, -b, -c)
    a + b + -b + -c = (a + -c) + (b + -b)
    b + -b = Real.0
    (a + -c) + (b + -b) = (a + -c) + Real.0
    (a + -c) + Real.0 = a + -c
    (a + b) - (b + c) = a + -c
    a - c = a + -c
    (a + b) - (b + c) = a - c
}

/// Subtracting a common term from two terms and then subtracting the
/// results gives the difference: `(x - z) - (y - z) = x - y`.
theorem triangle_real_sub_sub_same_base(x: Real, y: Real, z: Real) {
    (x - z) - (y - z) = x - y
} by {
    (x - z) - (y - z) = (x - z) + -(y - z)
    -(y - z) = -y + z
    (x - z) + -(y - z) = (x - z) + (-y + z)
    (x - z) + (-y + z) = x + -z + -y + z
    point2_add_four_last_next_to_first(x, -z, -y, z)
    x + -z + -y + z = (x + z) + (-z + -y)
    -z + -y = -(z + y)
    (x + z) + (-z + -y) = (x + z) - (z + y)
    triangle_real_add_sub_cancel_middle(x, y, z)
    (x + z) - (z + y) = x - y
    (x - z) - (y - z) = x - y
}

/// Subtracting two terms from a common term and then subtracting the
/// results gives the reversed difference: `(x - y) - (x - z) = z - y`.
theorem triangle_real_sub_sub_same_left(x: Real, y: Real, z: Real) {
    (x - y) - (x - z) = z - y
} by {
    (x - y) - (x - z) = (x - y) + -(x - z)
    -(x - z) = -x + z
    (x - y) + -(x - z) = (x - y) + (-x + z)
    (x - y) + (-x + z) = x + -y + -x + z
    point2_add_four_last_next_to_first(x, -y, -x, z)
    x + -y + -x + z = (x + z) + (-y + -x)
    -y + -x = -(y + x)
    y + x = x + y
    -(y + x) = -(x + y)
    (x + z) + (-y + -x) = (x + z) - (x + y)
    triangle_real_add_sub_cancel_middle(z, y, x)
    (z + x) - (x + y) = z - y
    x + z = z + x
    (x + z) - (x + y) = (z + x) - (x + y)
    (x + z) - (x + y) = z - y
    (x - y) - (x - z) = z - y
}

/// A nonzero scalar times a real is zero only if the real is zero:
/// `b * x = 0` and `b != 0` imply `x = 0`.
theorem triangle_mul_zero_cancel(b: Real, x: Real) {
    b * x = Real.0 and b != Real.0 implies x = Real.0
} by {
    if b * x = Real.0 and b != Real.0 {
        Real.0 = b * x
        mul_left_cancel(Real.0, b, x)
        Real.0 / b = x
        Real.0 / b = Real.0
        x = Real.0
    }
}

/// Subtracting then adding the same point on the right cancels:
/// `(x - c) + c = x`.
theorem triangle_sub_add_right_cancel(p: Point2[Real], q: Point2[Real]) {
    p.sub(q).add(q) = p
} by {
    point2_add_comm(p.sub(q), q)
    p.sub(q).add(q) = q.add(p.sub(q))
    point2_add_sub_left_cancel(q, p)
    q.add(p.sub(q)) = p
}

/// Adding then subtracting the same point on the right cancels:
/// `(x + y) - y = x`.
theorem triangle_add_sub_right_cancel(p: Point2[Real], q: Point2[Real]) {
    p.add(q).sub(q) = p
} by {
    p.add(q).sub(q).x = (p.x + q.x) - q.x
    (p.x + q.x) - q.x = p.x
    p.add(q).sub(q).x = p.x
    p.add(q).sub(q).y = (p.y + q.y) - q.y
    (p.y + q.y) - q.y = p.y
    p.add(q).sub(q).y = p.y
    point2_ext(p.add(q).sub(q), p)
}

/// Subtracting two points successively may be done in either order:
/// `(x - y) - z = (x - z) - y`.
theorem triangle_sub_sub_swap(p: Point2[Real], q: Point2[Real], r: Point2[Real]) {
    p.sub(q).sub(r) = p.sub(r).sub(q)
} by {
    p.sub(q).sub(r).x = (p.x - q.x) - r.x
    (p.x - q.x) - r.x = (p.x - r.x) - q.x
    p.sub(r).sub(q).x = (p.x - r.x) - q.x
    p.sub(q).sub(r).x = p.sub(r).sub(q).x
    p.sub(q).sub(r).y = (p.y - q.y) - r.y
    (p.y - q.y) - r.y = (p.y - r.y) - q.y
    p.sub(r).sub(q).y = (p.y - r.y) - q.y
    p.sub(q).sub(r).y = p.sub(r).sub(q).y
    point2_ext(p.sub(q).sub(r), p.sub(r).sub(q))
}

/// Subtracting a common point preserves equality: if `x = y` then
/// `x - z = y - z`.
theorem triangle_sub_cong_left(x: Point2[Real], y: Point2[Real], z: Point2[Real]) {
    x = y implies x.sub(z) = y.sub(z)
} by {
    if x = y {
        x.sub(z) = y.sub(z)
    }
}

/// The doubled displacement from one point to another is the difference of
/// the doubled points: `(m - g) + (m - g) = (m + m) - (g + g)`.
theorem triangle_double_sub_distrib(m: Point2[Real], g: Point2[Real]) {
    m.sub(g).add(m.sub(g)) = m.add(m).sub(g.add(g))
} by {
    point2_sub_add_sub(m, m, g, g)
    m.add(m).sub(g.add(g)) = m.sub(g).add(m.sub(g))
}

// ---------------------------------------------------------------------------
// The centroid and the median trisection.
// ---------------------------------------------------------------------------

/// The centroid of a triangle with vertices `a`, `b`, `c`: the point `g`
/// with `3g = a + b + c`.
define triangle_centroid(
    a: Point2[Real], b: Point2[Real], c: Point2[Real], g: Point2[Real]
) -> Bool {
    a.add(b).add(c) = g.add(g).add(g)
}

/// The defining equation of the centroid predicate.
theorem triangle_centroid_eq(
    a: Point2[Real], b: Point2[Real], c: Point2[Real], g: Point2[Real]
) {
    triangle_centroid(a, b, c, g) = (a.add(b).add(c) = g.add(g).add(g))
} by { }

/// The midpoint of a segment: the point `m` with `2m = a + b`.
define triangle_midpoint(a: Point2[Real], b: Point2[Real], m: Point2[Real]) -> Bool {
    a.add(b) = m.add(m)
}

/// The defining equation of the midpoint predicate.
theorem triangle_midpoint_eq(a: Point2[Real], b: Point2[Real], m: Point2[Real]) {
    triangle_midpoint(a, b, m) = (a.add(b) = m.add(m))
} by { }

/// With `m` the midpoint of `b` and `c` and `g` the centroid of `a`, `b`,
/// `c`, the doubled-sum identity `3g = a + 2m` holds.
theorem triangle_centroid_double_sum(
    a: Point2[Real], b: Point2[Real], c: Point2[Real],
    g: Point2[Real], m: Point2[Real]
) {
    triangle_midpoint(b, c, m) and triangle_centroid(a, b, c, g)
    implies
    g.add(g).add(g) = a.add(m.add(m))
} by {
    if triangle_midpoint(b, c, m) and triangle_centroid(a, b, c, g) {
        triangle_midpoint_eq(b, c, m)
        b.add(c) = m.add(m)
        triangle_centroid_eq(a, b, c, g)
        a.add(b).add(c) = g.add(g).add(g)
        point2_add_assoc(a, b, c)
        a.add(b).add(c) = a.add(b.add(c))
        a.add(b.add(c)) = a.add(m.add(m))
        a.add(b).add(c) = a.add(m.add(m))
        g.add(g).add(g) = a.add(m.add(m))
    }
}

/// From `3g = a + 2m` the median trisection follows: `g - a = 2 (m - g)`.
theorem triangle_trisection_of_double_sum(
    a: Point2[Real], m: Point2[Real], g: Point2[Real]
) {
    g.add(g).add(g) = a.add(m.add(m))
    implies
    g.sub(a) = m.sub(g).add(m.sub(g))
} by {
    if g.add(g).add(g) = a.add(m.add(m)) {
        // g - a = (3g - a) - 2g = (a + 2m - a) - 2g = 2m - 2g
        triangle_sub_cong_left(g.add(g).add(g), a.add(m.add(m)), a)
        g.add(g).add(g).sub(a) = a.add(m.add(m)).sub(a)
        point2_add_comm(a, m.add(m))
        a.add(m.add(m)) = m.add(m).add(a)
        triangle_add_sub_right_cancel(m.add(m), a)
        m.add(m).add(a).sub(a) = m.add(m)
        a.add(m.add(m)).sub(a) = m.add(m)
        g.add(g).add(g).sub(a) = m.add(m)
        triangle_sub_cong_left(g.add(g).add(g).sub(a), m.add(m), g.add(g))
        g.add(g).add(g).sub(a).sub(g.add(g)) = m.add(m).sub(g.add(g))
        // (3g - a) - 2g = (3g - 2g) - a = g - a
        triangle_sub_sub_swap(g.add(g).add(g), a, g.add(g))
        g.add(g).add(g).sub(a).sub(g.add(g)) =
            g.add(g).add(g).sub(g.add(g)).sub(a)
        point2_add_assoc(g, g, g)
        g.add(g).add(g) = g.add(g.add(g))
        triangle_add_sub_right_cancel(g, g.add(g))
        g.add(g.add(g)).sub(g.add(g)) = g
        g.add(g).add(g).sub(g.add(g)) = g
        g.add(g).add(g).sub(a).sub(g.add(g)) = g.sub(a)
        m.add(m).sub(g.add(g)) = g.sub(a)
        triangle_double_sub_distrib(m, g)
        m.sub(g).add(m.sub(g)) = m.add(m).sub(g.add(g))
        m.add(m).sub(g.add(g)) = m.sub(g).add(m.sub(g))
        g.sub(a) = m.sub(g).add(m.sub(g))
    }
}

/// The median trisection property of the centroid: with `m` the midpoint of
/// `b` and `c` and `g` the centroid of `a`, `b`, `c`, the displacement from
/// `a` to `g` is twice the displacement from `g` to `m`.  Hence `g` lies on
/// the median from `a` and divides it in the ratio `2 : 1`.
theorem triangle_centroid_median_trisection(
    a: Point2[Real], b: Point2[Real], c: Point2[Real],
    g: Point2[Real], m: Point2[Real]
) {
    triangle_midpoint(b, c, m) and triangle_centroid(a, b, c, g)
    implies
    g.sub(a) = m.sub(g).add(m.sub(g))
} by {
    if triangle_midpoint(b, c, m) and triangle_centroid(a, b, c, g) {
        triangle_centroid_double_sum(a, b, c, g, m)
        g.add(g).add(g) = a.add(m.add(m))
        triangle_trisection_of_double_sum(a, m, g)
        g.sub(a) = m.sub(g).add(m.sub(g))
    }
}

/// If `g - a = 2 (m - g)` then `a`, `g`, `m` are collinear: the centroid
/// lies on the median from `a`.
theorem triangle_trisection_collinear(
    a: Point2[Real], m: Point2[Real], g: Point2[Real]
) {
    g.sub(a) = m.sub(g).add(m.sub(g))
    implies
    a.collinear(g, m)
} by {
    if g.sub(a) = m.sub(g).add(m.sub(g)) {
        // m - a = (g - a) + (m - g)
        point2_sub_through(a, g, m)
        g.sub(a).add(m.sub(g)) = m.sub(a)
        // cross(g - a, m - a) = cross(g - a, (g - a) + (m - g))
        //                      = cross(g - a, m - g)
        point2_cross_add_right(g.sub(a), g.sub(a), m.sub(g))
        g.sub(a).cross(g.sub(a).add(m.sub(g))) =
            g.sub(a).cross(g.sub(a)) + g.sub(a).cross(m.sub(g))
        point2_cross_self(g.sub(a))
        g.sub(a).cross(g.sub(a)) = Real.0
        g.sub(a).cross(g.sub(a).add(m.sub(g))) = Real.0 + g.sub(a).cross(m.sub(g))
        Real.0 + g.sub(a).cross(m.sub(g)) = g.sub(a).cross(m.sub(g))
        g.sub(a).cross(g.sub(a).add(m.sub(g))) = g.sub(a).cross(m.sub(g))
        // g - a = (m - g) + (m - g), so cross(g - a, m - g) = 0
        point2_cross_add_left(m.sub(g), m.sub(g), m.sub(g))
        m.sub(g).add(m.sub(g)).cross(m.sub(g)) =
            m.sub(g).cross(m.sub(g)) + m.sub(g).cross(m.sub(g))
        point2_cross_self(m.sub(g))
        m.sub(g).cross(m.sub(g)) = Real.0
        m.sub(g).add(m.sub(g)).cross(m.sub(g)) = Real.0
        g.sub(a).cross(m.sub(g)) = Real.0
        g.sub(a).cross(g.sub(a).add(m.sub(g))) = Real.0
        g.sub(a).cross(m.sub(a)) = Real.0
        a.orientation(g, m) = g.sub(a).cross(m.sub(a))
        a.orientation(g, m) = Real.0
        a.collinear(g, m)
    }
}

/// The centroid lies on the median from `a`: the points `a`, `g`, `m` are
/// collinear with `m` the midpoint of `bc` and `g` the centroid.
theorem triangle_centroid_on_median(
    a: Point2[Real], b: Point2[Real], c: Point2[Real],
    g: Point2[Real], m: Point2[Real]
) {
    triangle_midpoint(b, c, m) and triangle_centroid(a, b, c, g)
    implies
    a.collinear(g, m)
} by {
    if triangle_midpoint(b, c, m) and triangle_centroid(a, b, c, g) {
        triangle_centroid_median_trisection(a, b, c, g, m)
        g.sub(a) = m.sub(g).add(m.sub(g))
        triangle_trisection_collinear(a, m, g)
        a.collinear(g, m)
    }
}

// ---------------------------------------------------------------------------
// The circumcenter and the perpendicular bisectors.
// ---------------------------------------------------------------------------

/// The circumcenter of a triangle: the point `o` equidistant from the three
/// vertices.
define triangle_circumcenter(
    o: Point2[Real], a: Point2[Real], b: Point2[Real], c: Point2[Real]
) -> Bool {
    a.dist_sq(o) = b.dist_sq(o) and b.dist_sq(o) = c.dist_sq(o)
}

/// The defining equation of the circumcenter predicate.
theorem triangle_circumcenter_eq(
    o: Point2[Real], a: Point2[Real], b: Point2[Real], c: Point2[Real]
) {
    triangle_circumcenter(o, a, b, c) =
        (a.dist_sq(o) = b.dist_sq(o) and b.dist_sq(o) = c.dist_sq(o))
} by { }

/// A triangle inscribed in the circle with center `o` and squared radius
/// `radius_sq`.
define triangle_circumcircle(
    o: Point2[Real], radius_sq: Real,
    a: Point2[Real], b: Point2[Real], c: Point2[Real]
) -> Bool {
    a.sub(o).norm_sq = radius_sq and
    b.sub(o).norm_sq = radius_sq and
    c.sub(o).norm_sq = radius_sq
}

/// The defining equation of the circumcircle predicate.
theorem triangle_circumcircle_eq(
    o: Point2[Real], radius_sq: Real,
    a: Point2[Real], b: Point2[Real], c: Point2[Real]
) {
    triangle_circumcircle(o, radius_sq, a, b, c) =
        (a.sub(o).norm_sq = radius_sq and
         b.sub(o).norm_sq = radius_sq and
         c.sub(o).norm_sq = radius_sq)
} by { }

/// The difference of the squared norms is the dot product of the difference
/// and the sum: `|p|^2 - |q|^2 = (p - q) . (p + q)`.
theorem triangle_norm_sq_sub_sq(p: Point2[Real], q: Point2[Real]) {
    p.norm_sq - q.norm_sq = p.sub(q).dot(p.add(q))
} by {
    p.sub(q).dot(p.add(q)) = p.dot(p.add(q)) - q.dot(p.add(q))
    point2_dot_sub_left(p, q, p.add(q))
    p.dot(p.add(q)) = p.dot(p) + p.dot(q)
    point2_dot_add_right(p, p, q)
    q.dot(p.add(q)) = q.dot(p) + q.dot(q)
    point2_dot_add_right(q, p, q)
    point2_dot_comm(q, p)
    q.dot(p) = p.dot(q)
    q.dot(p) + q.dot(q) = p.dot(q) + q.dot(q)
    q.dot(p.add(q)) = p.dot(q) + q.dot(q)
    p.dot(p.add(q)) - q.dot(p.add(q)) = (p.dot(p) + p.dot(q)) - (p.dot(q) + q.dot(q))
    triangle_real_add_sub_cancel_middle(p.dot(p), p.dot(q), q.dot(q))
    (p.dot(p) + p.dot(q)) - (p.dot(q) + q.dot(q)) = p.dot(p) - q.dot(q)
    p.dot(p) = p.norm_sq
    q.dot(q) = q.norm_sq
    p.dot(p) - q.dot(q) = p.norm_sq - q.norm_sq
    p.sub(q).dot(p.add(q)) = p.norm_sq - q.norm_sq
    p.norm_sq - q.norm_sq = p.sub(q).dot(p.add(q))
}

/// Subtracting a common point from two points and then subtracting the
/// results gives the difference of the points: `(x - z) - (y - z) = x - y`.
theorem triangle_sub_sub_same_base(
    x: Point2[Real], y: Point2[Real], z: Point2[Real]
) {
    x.sub(z).sub(y.sub(z)) = x.sub(y)
} by {
    x.sub(z).sub(y.sub(z)).x = (x.x - z.x) - (y.x - z.x)
    triangle_real_sub_sub_same_base(x.x, y.x, z.x)
    (x.x - z.x) - (y.x - z.x) = x.x - y.x
    x.sub(y).x = x.x - y.x
    x.sub(z).sub(y.sub(z)).x = x.sub(y).x
    x.sub(z).sub(y.sub(z)).y = (x.y - z.y) - (y.y - z.y)
    triangle_real_sub_sub_same_base(x.y, y.y, z.y)
    (x.y - z.y) - (y.y - z.y) = x.y - y.y
    x.sub(y).y = x.y - y.y
    x.sub(z).sub(y.sub(z)).y = x.sub(y).y
    point2_ext(x.sub(z).sub(y.sub(z)), x.sub(y))
}

/// Subtracting two points from a common point and then subtracting the
/// results gives the reversed difference: `(x - y) - (x - z) = z - y`.
theorem triangle_sub_sub_same_left(
    x: Point2[Real], y: Point2[Real], z: Point2[Real]
) {
    x.sub(y).sub(x.sub(z)) = z.sub(y)
} by {
    x.sub(y).sub(x.sub(z)).x = (x.x - y.x) - (x.x - z.x)
    triangle_real_sub_sub_same_left(x.x, y.x, z.x)
    (x.x - y.x) - (x.x - z.x) = z.x - y.x
    z.sub(y).x = z.x - y.x
    x.sub(y).sub(x.sub(z)).x = z.sub(y).x
    x.sub(y).sub(x.sub(z)).y = (x.y - y.y) - (x.y - z.y)
    triangle_real_sub_sub_same_left(x.y, y.y, z.y)
    (x.y - y.y) - (x.y - z.y) = z.y - y.y
    z.sub(y).y = z.y - y.y
    x.sub(y).sub(x.sub(z)).y = z.sub(y).y
    point2_ext(x.sub(y).sub(x.sub(z)), z.sub(y))
}

/// A point equidistant from two vertices lies on the perpendicular bisector
/// of the side between them: with `2m = a + b` and `|o - a|^2 = |o - b|^2`,
/// the displacement `o - m` is perpendicular to `a - b`.
theorem triangle_equidistant_perp_bisector_dot(
    o: Point2[Real], a: Point2[Real], b: Point2[Real], m: Point2[Real]
) {
    triangle_midpoint(a, b, m) and a.dist_sq(o) = b.dist_sq(o)
    implies
    o.sub(m).dot(a.sub(b)) = Real.0
} by {
    if triangle_midpoint(a, b, m) and a.dist_sq(o) = b.dist_sq(o) {
        triangle_midpoint_eq(a, b, m)
        a.add(b) = m.add(m)
        a.dist_sq(o) = b.dist_sq(o)
        // |a - o|^2 - |b - o|^2 = 0
        a.dist_sq(o) - b.dist_sq(o) = Real.0
        point2_dist_sq_eq_norm_sq_sub(a, o)
        a.dist_sq(o) = a.sub(o).norm_sq
        point2_dist_sq_eq_norm_sq_sub(b, o)
        b.dist_sq(o) = b.sub(o).norm_sq
        a.sub(o).norm_sq - b.sub(o).norm_sq = Real.0
        // difference of squares:
        //   |a - o|^2 - |b - o|^2 = ((a - o) - (b - o)) . ((a - o) + (b - o))
        triangle_norm_sq_sub_sq(a.sub(o), b.sub(o))
        a.sub(o).norm_sq - b.sub(o).norm_sq =
            a.sub(o).sub(b.sub(o)).dot(a.sub(o).add(b.sub(o)))
        a.sub(o).sub(b.sub(o)).dot(a.sub(o).add(b.sub(o))) = Real.0
        // (a - o) - (b - o) = a - b
        triangle_sub_sub_same_base(a, b, o)
        a.sub(o).sub(b.sub(o)) = a.sub(b)
        a.sub(b).dot(a.sub(o).add(b.sub(o))) = Real.0
        // (a - o) + (b - o) = (a + b) - (o + o) = (m + m) - (o + o)
        //                    = (m - o) + (m - o)
        point2_sub_add_sub(a, b, o, o)
        a.add(b).sub(o.add(o)) = a.sub(o).add(b.sub(o))
        a.sub(o).add(b.sub(o)) = a.add(b).sub(o.add(o))
        a.sub(o).add(b.sub(o)) = m.add(m).sub(o.add(o))
        triangle_double_sub_distrib(m, o)
        m.sub(o).add(m.sub(o)) = m.add(m).sub(o.add(o))
        m.add(m).sub(o.add(o)) = m.sub(o).add(m.sub(o))
        a.sub(o).add(b.sub(o)) = m.sub(o).add(m.sub(o))
        // so (a - b) . ((m - o) + (m - o)) = 0
        a.sub(b).dot(m.sub(o).add(m.sub(o))) = Real.0
        point2_dot_add_right(a.sub(b), m.sub(o), m.sub(o))
        a.sub(b).dot(m.sub(o).add(m.sub(o))) =
            a.sub(b).dot(m.sub(o)) + a.sub(b).dot(m.sub(o))
        a.sub(b).dot(m.sub(o)) + a.sub(b).dot(m.sub(o)) = Real.0
        (Real.1 + Real.1) * (a.sub(b).dot(m.sub(o))) = Real.0
        triangle_double_zero_cancel(a.sub(b).dot(m.sub(o)))
        a.sub(b).dot(m.sub(o)) = Real.0
        // m - o = -(o - m), so (a - b) . (o - m) = 0
        point2_sub_reverse_neg(o, m)
        o.sub(m) = m.sub(o).neg
        point2_dot_neg_right(a.sub(b), m.sub(o))
        a.sub(b).dot(m.sub(o).neg) = -a.sub(b).dot(m.sub(o))
        a.sub(b).dot(o.sub(m)) = -a.sub(b).dot(m.sub(o))
        -a.sub(b).dot(m.sub(o)) = Real.0
        a.sub(b).dot(o.sub(m)) = Real.0
        point2_dot_comm(a.sub(b), o.sub(m))
        o.sub(m).dot(a.sub(b)) = Real.0
    }
}

/// A circumcenter of a triangle lies on the perpendicular bisector of each
/// side: with `m_ab`, `m_bc`, `m_ca` the side midpoints, the displacements
/// from the circumcenter `o` to the midpoints are perpendicular to the
/// respective sides.
theorem triangle_circumcenter_perp_bisectors(
    o: Point2[Real], a: Point2[Real], b: Point2[Real], c: Point2[Real],
    m_ab: Point2[Real], m_bc: Point2[Real], m_ca: Point2[Real]
) {
    triangle_circumcenter(o, a, b, c) and
    triangle_midpoint(a, b, m_ab) and
    triangle_midpoint(b, c, m_bc) and
    triangle_midpoint(c, a, m_ca)
    implies
    o.sub(m_ab).dot(a.sub(b)) = Real.0 and
    o.sub(m_bc).dot(b.sub(c)) = Real.0 and
    o.sub(m_ca).dot(c.sub(a)) = Real.0
} by {
    if triangle_circumcenter(o, a, b, c) and
        triangle_midpoint(a, b, m_ab) and
        triangle_midpoint(b, c, m_bc) and
        triangle_midpoint(c, a, m_ca) {
        triangle_circumcenter_eq(o, a, b, c)
        a.dist_sq(o) = b.dist_sq(o) and b.dist_sq(o) = c.dist_sq(o)
        a.dist_sq(o) = b.dist_sq(o)
        b.dist_sq(o) = c.dist_sq(o)
        triangle_midpoint_eq(a, b, m_ab)
        a.add(b) = m_ab.add(m_ab)
        triangle_equidistant_perp_bisector_dot(o, a, b, m_ab)
        o.sub(m_ab).dot(a.sub(b)) = Real.0
        // side bc
        triangle_midpoint_eq(b, c, m_bc)
        b.add(c) = m_bc.add(m_bc)
        point2_dist_sq_comm(b, o)
        triangle_equidistant_perp_bisector_dot(o, b, c, m_bc)
        o.sub(m_bc).dot(b.sub(c)) = Real.0
        // side ca
        triangle_midpoint_eq(c, a, m_ca)
        c.add(a) = m_ca.add(m_ca)
        triangle_equidistant_perp_bisector_dot(o, c, a, m_ca)
        o.sub(m_ca).dot(c.sub(a)) = Real.0
        o.sub(m_ab).dot(a.sub(b)) = Real.0 and
            o.sub(m_bc).dot(b.sub(c)) = Real.0 and
            o.sub(m_ca).dot(c.sub(a)) = Real.0
    }
}

// ---------------------------------------------------------------------------
// The orthocenter and the altitudes.
// ---------------------------------------------------------------------------

/// The orthocenter of a triangle: the point `h` where the three altitudes
/// meet, i.e. the point whose displacements from the vertices are
/// perpendicular to the opposite sides.
define triangle_orthocenter(
    h: Point2[Real], a: Point2[Real], b: Point2[Real], c: Point2[Real]
) -> Bool {
    h.sub(a).dot(b.sub(c)) = Real.0 and
    h.sub(b).dot(c.sub(a)) = Real.0 and
    h.sub(c).dot(a.sub(b)) = Real.0
}

/// The defining equation of the orthocenter predicate.
theorem triangle_orthocenter_eq(
    h: Point2[Real], a: Point2[Real], b: Point2[Real], c: Point2[Real]
) {
    triangle_orthocenter(h, a, b, c) =
        (h.sub(a).dot(b.sub(c)) = Real.0 and
         h.sub(b).dot(c.sub(a)) = Real.0 and
         h.sub(c).dot(a.sub(b)) = Real.0)
} by { }

/// A three-point sum may be cyclically rotated: `a + b + c = b + c + a`.
theorem triangle_add_three_rotate(
    a: Point2[Real], b: Point2[Real], c: Point2[Real]
) {
    a.add(b).add(c) = b.add(c).add(a)
} by {
    point2_add_comm(a, b)
    a.add(b) = b.add(a)
    a.add(b).add(c) = b.add(a).add(c)
    point2_add_assoc(b, a, c)
    b.add(a).add(c) = b.add(a.add(c))
    point2_add_comm(a, c)
    a.add(c) = c.add(a)
    b.add(a.add(c)) = b.add(c.add(a))
    point2_add_assoc(b, c, a)
    b.add(c.add(a)) = b.add(c).add(a)
    a.add(b).add(c) = b.add(c).add(a)
}

/// With `h = a + b + c - 2o` and `a`, `b` on the circle centered at `o`, the
/// altitude through `c` is perpendicular to the side `ab`:
/// `(h - c) . (a - b) = 0`.
theorem triangle_euler_altitude_dot(
    o: Point2[Real], radius_sq: Real,
    a: Point2[Real], b: Point2[Real], c: Point2[Real],
    h: Point2[Real]
) {
    a.sub(o).norm_sq = radius_sq and
    b.sub(o).norm_sq = radius_sq and
    a.add(b).add(c).sub(o.add(o)) = h
    implies
    h.sub(c).dot(a.sub(b)) = Real.0
} by {
    if a.sub(o).norm_sq = radius_sq and
        b.sub(o).norm_sq = radius_sq and
        a.add(b).add(c).sub(o.add(o)) = h {
        a.sub(o).norm_sq = radius_sq
        b.sub(o).norm_sq = radius_sq
        a.add(b).add(c).sub(o.add(o)) = h
        // h - c = ((a + b + c) - 2o) - c = (a + b + c - c) - 2o = (a + b) - 2o
        h.sub(c) = a.add(b).add(c).sub(o.add(o)).sub(c)
        triangle_sub_sub_swap(a.add(b).add(c), o.add(o), c)
        a.add(b).add(c).sub(o.add(o)).sub(c) =
            a.add(b).add(c).sub(c).sub(o.add(o))
        triangle_add_sub_right_cancel(a.add(b), c)
        a.add(b).add(c).sub(c) = a.add(b)
        a.add(b).add(c).sub(o.add(o)).sub(c) = a.add(b).sub(o.add(o))
        h.sub(c) = a.add(b).sub(o.add(o))
        point2_sub_add_sub(a, b, o, o)
        a.add(b).sub(o.add(o)) = a.sub(o).add(b.sub(o))
        h.sub(c) = a.sub(o).add(b.sub(o))
        // a - b = (a - o) - (b - o)
        triangle_sub_sub_same_base(a, b, o)
        a.sub(o).sub(b.sub(o)) = a.sub(b)
        // (u + v) . (u - v) = |u|^2 - |v|^2 with u = a - o, v = b - o
        triangle_norm_sq_sub_sq(a.sub(o), b.sub(o))
        a.sub(o).norm_sq - b.sub(o).norm_sq =
            a.sub(o).sub(b.sub(o)).dot(a.sub(o).add(b.sub(o)))
        point2_dot_comm(a.sub(o).add(b.sub(o)), a.sub(o).sub(b.sub(o)))
        a.sub(o).add(b.sub(o)).dot(a.sub(o).sub(b.sub(o))) =
            a.sub(o).sub(b.sub(o)).dot(a.sub(o).add(b.sub(o)))
        a.sub(o).add(b.sub(o)).dot(a.sub(o).sub(b.sub(o))) =
            a.sub(o).norm_sq - b.sub(o).norm_sq
        a.sub(o).norm_sq - b.sub(o).norm_sq = radius_sq - radius_sq
        radius_sq - radius_sq = Real.0
        a.sub(o).add(b.sub(o)).dot(a.sub(o).sub(b.sub(o))) = Real.0
        h.sub(c).dot(a.sub(b)) = Real.0
    }
}

/// The point `h = a + b + c - 2o` for a circumcenter `o` is the orthocenter:
/// it lies on all three altitudes.
theorem triangle_orthocenter_altitudes(
    o: Point2[Real], radius_sq: Real,
    a: Point2[Real], b: Point2[Real], c: Point2[Real],
    h: Point2[Real]
) {
    a.sub(o).norm_sq = radius_sq and
    b.sub(o).norm_sq = radius_sq and
    c.sub(o).norm_sq = radius_sq and
    a.add(b).add(c).sub(o.add(o)) = h
    implies
    h.sub(a).dot(b.sub(c)) = Real.0 and
    h.sub(b).dot(c.sub(a)) = Real.0 and
    h.sub(c).dot(a.sub(b)) = Real.0
} by {
    if a.sub(o).norm_sq = radius_sq and
        b.sub(o).norm_sq = radius_sq and
        c.sub(o).norm_sq = radius_sq and
        a.add(b).add(c).sub(o.add(o)) = h {
        a.sub(o).norm_sq = radius_sq
        b.sub(o).norm_sq = radius_sq
        c.sub(o).norm_sq = radius_sq
        a.add(b).add(c).sub(o.add(o)) = h
        // altitude from a: side bc
        triangle_add_three_rotate(a, b, c)
        a.add(b).add(c) = b.add(c).add(a)
        triangle_sub_cong_left(a.add(b).add(c), b.add(c).add(a), o.add(o))
        a.add(b).add(c).sub(o.add(o)) = b.add(c).add(a).sub(o.add(o))
        b.add(c).add(a).sub(o.add(o)) = h
        triangle_euler_altitude_dot(o, radius_sq, b, c, a, h)
        h.sub(a).dot(b.sub(c)) = Real.0
        // altitude from b: side ca
        triangle_add_three_rotate(b, c, a)
        b.add(c).add(a) = c.add(a).add(b)
        triangle_sub_cong_left(b.add(c).add(a), c.add(a).add(b), o.add(o))
        b.add(c).add(a).sub(o.add(o)) = c.add(a).add(b).sub(o.add(o))
        c.add(a).add(b).sub(o.add(o)) = h
        triangle_euler_altitude_dot(o, radius_sq, c, a, b, h)
        h.sub(b).dot(c.sub(a)) = Real.0
        // altitude from c: side ab
        triangle_euler_altitude_dot(o, radius_sq, a, b, c, h)
        h.sub(c).dot(a.sub(b)) = Real.0
        h.sub(a).dot(b.sub(c)) = Real.0 and
            h.sub(b).dot(c.sub(a)) = Real.0 and
            h.sub(c).dot(a.sub(b)) = Real.0
    }
}

// ---------------------------------------------------------------------------
// The angle bisector theorem.
// ---------------------------------------------------------------------------

/// Scalar multiplication distributes over point addition:
/// `k (p + q) = k p + k q`.
theorem triangle_smul_add_distrib(p: Point2[Real], q: Point2[Real], k: Real) {
    p.add(q).smul(k) = p.smul(k).add(q.smul(k))
} by {
    p.add(q).smul(k).x = k * (p.x + q.x)
    k * (p.x + q.x) = k * p.x + k * q.x
    p.smul(k).add(q.smul(k)).x = p.smul(k).x + q.smul(k).x
    p.smul(k).x = k * p.x
    q.smul(k).x = k * q.x
    p.smul(k).add(q.smul(k)).x = k * p.x + k * q.x
    p.add(q).smul(k).x = p.smul(k).add(q.smul(k)).x
    p.add(q).smul(k).y = k * (p.y + q.y)
    k * (p.y + q.y) = k * p.y + k * q.y
    p.smul(k).add(q.smul(k)).y = p.smul(k).y + q.smul(k).y
    p.smul(k).y = k * p.y
    q.smul(k).y = k * q.y
    p.smul(k).add(q.smul(k)).y = k * p.y + k * q.y
    p.add(q).smul(k).y = p.smul(k).add(q.smul(k)).y
    point2_ext(p.add(q).smul(k), p.smul(k).add(q.smul(k)))
}

/// Scalar multiplication distributes over point subtraction:
/// `k (p - q) = k p - k q`.
theorem triangle_smul_sub_distrib(p: Point2[Real], q: Point2[Real], k: Real) {
    p.sub(q).smul(k) = p.smul(k).sub(q.smul(k))
} by {
    p.sub(q).smul(k).x = k * (p.x - q.x)
    k * (p.x - q.x) = k * p.x - k * q.x
    p.smul(k).sub(q.smul(k)).x = p.smul(k).x - q.smul(k).x
    p.smul(k).x = k * p.x
    q.smul(k).x = k * q.x
    p.smul(k).sub(q.smul(k)).x = k * p.x - k * q.x
    p.sub(q).smul(k).x = p.smul(k).sub(q.smul(k)).x
    p.sub(q).smul(k).y = k * (p.y - q.y)
    k * (p.y - q.y) = k * p.y - k * q.y
    p.smul(k).sub(q.smul(k)).y = p.smul(k).y - q.smul(k).y
    p.smul(k).y = k * p.y
    q.smul(k).y = k * q.y
    p.smul(k).sub(q.smul(k)).y = k * p.y - k * q.y
    p.sub(q).smul(k).y = p.smul(k).sub(q.smul(k)).y
    point2_ext(p.sub(q).smul(k), p.smul(k).sub(q.smul(k)))
}

/// Scalar multiplication distributes over scalar addition:
/// `k1 p + k2 p = (k1 + k2) p`.
theorem triangle_smul_add_scalar(p: Point2[Real], k1: Real, k2: Real) {
    p.smul(k1).add(p.smul(k2)) = p.smul(k1 + k2)
} by {
    p.smul(k1).add(p.smul(k2)).x = p.smul(k1).x + p.smul(k2).x
    p.smul(k1).x = k1 * p.x
    p.smul(k2).x = k2 * p.x
    p.smul(k1).add(p.smul(k2)).x = k1 * p.x + k2 * p.x
    p.smul(k1 + k2).x = (k1 + k2) * p.x
    k1 * p.x + k2 * p.x = (k1 + k2) * p.x
    p.smul(k1).add(p.smul(k2)).x = p.smul(k1 + k2).x
    p.smul(k1).add(p.smul(k2)).y = p.smul(k1).y + p.smul(k2).y
    p.smul(k1).y = k1 * p.y
    p.smul(k2).y = k2 * p.y
    p.smul(k1).add(p.smul(k2)).y = k1 * p.y + k2 * p.y
    p.smul(k1 + k2).y = (k1 + k2) * p.y
    k1 * p.y + k2 * p.y = (k1 + k2) * p.y
    p.smul(k1).add(p.smul(k2)).y = p.smul(k1 + k2).y
    point2_ext(p.smul(k1).add(p.smul(k2)), p.smul(k1 + k2))
}

/// Scalar multiplication distributes over scalar subtraction:
/// `k1 p - k2 p = (k1 - k2) p`.
theorem triangle_smul_sub_scalar(p: Point2[Real], k1: Real, k2: Real) {
    p.smul(k1).sub(p.smul(k2)) = p.smul(k1 - k2)
} by {
    p.smul(k1).sub(p.smul(k2)).x = p.smul(k1).x - p.smul(k2).x
    p.smul(k1).x = k1 * p.x
    p.smul(k2).x = k2 * p.x
    p.smul(k1).sub(p.smul(k2)).x = k1 * p.x - k2 * p.x
    p.smul(k1 - k2).x = (k1 - k2) * p.x
    k1 * p.x - k2 * p.x = (k1 - k2) * p.x
    p.smul(k1).sub(p.smul(k2)).x = p.smul(k1 - k2).x
    p.smul(k1).sub(p.smul(k2)).y = p.smul(k1).y - p.smul(k2).y
    p.smul(k1).y = k1 * p.y
    p.smul(k2).y = k2 * p.y
    p.smul(k1).sub(p.smul(k2)).y = k1 * p.y - k2 * p.y
    p.smul(k1 - k2).y = (k1 - k2) * p.y
    k1 * p.y - k2 * p.y = (k1 - k2) * p.y
    p.smul(k1).sub(p.smul(k2)).y = p.smul(k1 - k2).y
    point2_ext(p.smul(k1).sub(p.smul(k2)), p.smul(k1 - k2))
}

/// A scalar multiple of a negation is the negation of the scalar multiple:
/// `(-k) p = -(k p)`.
theorem triangle_smul_neg_scalar(p: Point2[Real], k: Real) {
    p.smul(-k) = p.smul(k).neg
} by {
    p.smul(-k).x = (-k) * p.x
    (-k) * p.x = -(k * p.x)
    p.smul(k).neg.x = -p.smul(k).x
    p.smul(k).x = k * p.x
    p.smul(k).neg.x = -(k * p.x)
    p.smul(-k).x = p.smul(k).neg.x
    p.smul(-k).y = (-k) * p.y
    (-k) * p.y = -(k * p.y)
    p.smul(k).neg.y = -p.smul(k).y
    p.smul(k).y = k * p.y
    p.smul(k).neg.y = -(k * p.y)
    p.smul(-k).y = p.smul(k).neg.y
    point2_ext(p.smul(-k), p.smul(k).neg)
}

/// A point minus `k` times itself is `(1 - k)` times itself:
/// `p - k p = (1 - k) p`.
theorem triangle_smul_sub_self(p: Point2[Real], k: Real) {
    p.sub(p.smul(k)) = p.smul(Real.1 - k)
} by {
    p.sub(p.smul(k)).x = p.x - k * p.x
    p.x - k * p.x = (Real.1 - k) * p.x
    p.smul(Real.1 - k).x = (Real.1 - k) * p.x
    p.sub(p.smul(k)).x = p.smul(Real.1 - k).x
    p.sub(p.smul(k)).y = p.y - k * p.y
    p.y - k * p.y = (Real.1 - k) * p.y
    p.smul(Real.1 - k).y = (Real.1 - k) * p.y
    p.sub(p.smul(k)).y = p.smul(Real.1 - k).y
    point2_ext(p.sub(p.smul(k)), p.smul(Real.1 - k))
}

/// A point plus `k` times itself is `(1 + k)` times itself:
/// `p + k p = (1 + k) p`.
theorem triangle_smul_add_self(p: Point2[Real], k: Real) {
    p.add(p.smul(k)) = p.smul(Real.1 + k)
} by {
    p.add(p.smul(k)).x = p.x + k * p.x
    p.x + k * p.x = (Real.1 + k) * p.x
    p.smul(Real.1 + k).x = (Real.1 + k) * p.x
    p.add(p.smul(k)).x = p.smul(Real.1 + k).x
    p.add(p.smul(k)).y = p.y + k * p.y
    p.y + k * p.y = (Real.1 + k) * p.y
    p.smul(Real.1 + k).y = (Real.1 + k) * p.y
    p.add(p.smul(k)).y = p.smul(Real.1 + k).y
    point2_ext(p.add(p.smul(k)), p.smul(Real.1 + k))
}

/// With `d = (1 - t) b + t c`, the displacement `d - a` is the affine
/// combination `(1 - t)(b - a) + t (c - a)`.
theorem triangle_angle_bisector_d_sub_a(
    a: Point2[Real], b: Point2[Real], c: Point2[Real],
    d: Point2[Real], t: Real
) {
    b.smul(Real.1 - t).add(c.smul(t)) = d
    implies
    d.sub(a) = b.sub(a).smul(Real.1 - t).add(c.sub(a).smul(t))
} by {
    if b.smul(Real.1 - t).add(c.smul(t)) = d {
        b.sub(a).smul(Real.1 - t).add(c.sub(a).smul(t)) =
            b.smul(Real.1 - t).sub(a.smul(Real.1 - t)).add(c.smul(t).sub(a.smul(t)))
        triangle_smul_sub_distrib(b, a, Real.1 - t)
        b.sub(a).smul(Real.1 - t) = b.smul(Real.1 - t).sub(a.smul(Real.1 - t))
        triangle_smul_sub_distrib(c, a, t)
        c.sub(a).smul(t) = c.smul(t).sub(a.smul(t))
        b.smul(Real.1 - t).sub(a.smul(Real.1 - t)).add(c.smul(t).sub(a.smul(t))) =
            b.smul(Real.1 - t).add(c.smul(t)).sub(a.smul(Real.1 - t).add(a.smul(t)))
        point2_sub_add_sub(b.smul(Real.1 - t), c.smul(t), a.smul(Real.1 - t), a.smul(t))
        b.sub(a).smul(Real.1 - t).add(c.sub(a).smul(t)) =
            b.smul(Real.1 - t).add(c.smul(t)).sub(a.smul(Real.1 - t).add(a.smul(t)))
        b.smul(Real.1 - t).add(c.smul(t)) = d
        b.sub(a).smul(Real.1 - t).add(c.sub(a).smul(t)) =
            d.sub(a.smul(Real.1 - t).add(a.smul(t)))
        triangle_smul_add_scalar(a, Real.1 - t, t)
        a.smul(Real.1 - t).add(a.smul(t)) = a.smul(Real.1 - t + t)
        Real.1 - t + t = Real.1
        a.smul(Real.1 - t + t) = a.smul(Real.1)
        point2_smul_one_left(a)
        a.smul(Real.1) = a
        a.smul(Real.1 - t + t) = a
        a.smul(Real.1 - t).add(a.smul(t)) = a
        d.sub(a.smul(Real.1 - t).add(a.smul(t))) = d.sub(a)
        b.sub(a).smul(Real.1 - t).add(c.sub(a).smul(t)) = d.sub(a)
        d.sub(a) = b.sub(a).smul(Real.1 - t).add(c.sub(a).smul(t))
    }
}

/// With `d = (1 - t) b + t c`, the displacement `d - b` is `t (c - b)`.
theorem triangle_angle_bisector_d_sub_b(
    b: Point2[Real], c: Point2[Real], d: Point2[Real], t: Real
) {
    b.smul(Real.1 - t).add(c.smul(t)) = d
    implies
    d.sub(b) = c.sub(b).smul(t)
} by {
    if b.smul(Real.1 - t).add(c.smul(t)) = d {
        d.sub(b) = b.smul(Real.1 - t).add(c.smul(t)).sub(b)
        point2_sub_add_sub(b.smul(Real.1 - t), c.smul(t), b, point2_zero[Real])
        b.smul(Real.1 - t).add(c.smul(t)).sub(b.add(point2_zero[Real])) =
            b.smul(Real.1 - t).sub(b).add(c.smul(t).sub(point2_zero[Real]))
        point2_add_zero_right(b)
        b.add(point2_zero[Real]) = b
        b.smul(Real.1 - t).add(c.smul(t)).sub(b) =
            b.smul(Real.1 - t).sub(b).add(c.smul(t).sub(point2_zero[Real]))
        point2_sub_zero_right(c.smul(t))
        c.smul(t).sub(point2_zero[Real]) = c.smul(t)
        b.smul(Real.1 - t).add(c.smul(t)).sub(b) =
            b.smul(Real.1 - t).sub(b).add(c.smul(t))
        // (1 - t) b - b = -t b
        triangle_smul_sub_self(b, Real.1 - t)
        b.sub(b.smul(Real.1 - t)) = b.smul(Real.1 - (Real.1 - t))
        b.smul(Real.1 - t).sub(b) = b.smul(Real.1 - t).sub(b)
        point2_sub_reverse_neg(b.smul(Real.1 - t), b)
        b.smul(Real.1 - t).sub(b) = b.sub(b.smul(Real.1 - t)).neg
        b.sub(b.smul(Real.1 - t)).neg = b.smul(Real.1 - (Real.1 - t)).neg
        Real.1 - (Real.1 - t) = Real.1 + -(Real.1 - t)
        -(Real.1 - t) = -Real.1 + t
        Real.1 + -(Real.1 - t) = Real.1 + (-Real.1 + t)
        Real.1 + (-Real.1 + t) = Real.1 + -Real.1 + t
        Real.1 + -Real.1 = Real.0
        Real.1 + -Real.1 + t = Real.0 + t
        Real.0 + t = t
        Real.1 - (Real.1 - t) = t
        b.smul(Real.1 - (Real.1 - t)) = b.smul(t)
        b.smul(Real.1 - (Real.1 - t)).neg = b.smul(t).neg
        b.smul(Real.1 - t).sub(b) = b.smul(t).neg
        b.smul(t).neg.add(c.smul(t)) = c.smul(t).sub(b.smul(t))
        point2_sub_eq_add_neg(c.smul(t), b.smul(t))
        b.smul(t).neg.add(c.smul(t)) = c.smul(t).sub(b.smul(t))
        b.smul(Real.1 - t).add(c.smul(t)).sub(b) = c.smul(t).sub(b.smul(t))
        triangle_smul_sub_distrib(c, b, t)
        c.sub(b).smul(t) = c.smul(t).sub(b.smul(t))
        b.smul(Real.1 - t).add(c.smul(t)).sub(b) = c.sub(b).smul(t)
        d.sub(b) = c.sub(b).smul(t)
    }
}

/// With `d = (1 - t) b + t c`, the displacement `d - c` is `(1 - t)(b - c)`.
theorem triangle_angle_bisector_d_sub_c(
    b: Point2[Real], c: Point2[Real], d: Point2[Real], t: Real
) {
    b.smul(Real.1 - t).add(c.smul(t)) = d
    implies
    d.sub(c) = b.sub(c).smul(Real.1 - t)
} by {
    if b.smul(Real.1 - t).add(c.smul(t)) = d {
        d.sub(c) = b.smul(Real.1 - t).add(c.smul(t)).sub(c)
        point2_sub_add_sub(b.smul(Real.1 - t), c.smul(t), c, point2_zero[Real])
        b.smul(Real.1 - t).add(c.smul(t)).sub(c.add(point2_zero[Real])) =
            b.smul(Real.1 - t).sub(c).add(c.smul(t).sub(point2_zero[Real]))
        point2_add_zero_right(c)
        c.add(point2_zero[Real]) = c
        b.smul(Real.1 - t).add(c.smul(t)).sub(c) =
            b.smul(Real.1 - t).sub(c).add(c.smul(t).sub(point2_zero[Real]))
        point2_sub_zero_right(c.smul(t))
        c.smul(t).sub(point2_zero[Real]) = c.smul(t)
        b.smul(Real.1 - t).add(c.smul(t)).sub(c) =
            b.smul(Real.1 - t).sub(c).add(c.smul(t))
        // t c - c = (t - 1) c = -(1 - t) c
        point2_sub_reverse_neg(c, c.smul(t))
        c.sub(c.smul(t)) = c.smul(t).sub(c).neg
        triangle_smul_sub_self(c, t)
        c.sub(c.smul(t)) = c.smul(Real.1 - t)
        c.smul(t).sub(c).neg = c.smul(Real.1 - t)
        c.smul(t).sub(c) = c.smul(Real.1 - t).neg
        point2_sub_eq_add_neg(c.smul(Real.1 - t), c.smul(t).sub(c))
        // (1 - t) b + (t c - c) = (1 - t) b - (1 - t) c
        b.smul(Real.1 - t).sub(c).add(c.smul(t)) =
            b.smul(Real.1 - t).add(c.smul(t)).sub(c)
        b.smul(Real.1 - t).add(c.smul(t)).sub(c) =
            b.smul(Real.1 - t).add(c.smul(t)).add(c.neg)
        point2_sub_eq_add_neg(b.smul(Real.1 - t).add(c.smul(t)), c)
        point2_add_assoc(b.smul(Real.1 - t), c.smul(t), c.neg)
        b.smul(Real.1 - t).add(c.smul(t)).add(c.neg) =
            b.smul(Real.1 - t).add(c.smul(t).add(c.neg))
        point2_sub_eq_add_neg(c.smul(t), c)
        c.smul(t).add(c.neg) = c.smul(t).sub(c)
        b.smul(Real.1 - t).add(c.smul(t).add(c.neg)) =
            b.smul(Real.1 - t).add(c.smul(t).sub(c))
        b.smul(Real.1 - t).add(c.smul(t)).sub(c) =
            b.smul(Real.1 - t).add(c.smul(t).sub(c))
        b.smul(Real.1 - t).add(c.smul(t).sub(c)) =
            b.smul(Real.1 - t).sub(c.smul(Real.1 - t))
        point2_sub_eq_add_neg(b.smul(Real.1 - t), c.smul(Real.1 - t))
        b.smul(Real.1 - t).add(c.smul(Real.1 - t).neg) =
            b.smul(Real.1 - t).sub(c.smul(Real.1 - t))
        b.smul(Real.1 - t).add(c.smul(Real.1 - t).neg) =
            b.smul(Real.1 - t).add(c.smul(t).sub(c))
        b.smul(Real.1 - t).add(c.smul(t).sub(c)) =
            b.smul(Real.1 - t).sub(c.smul(Real.1 - t))
        d.sub(c) = b.smul(Real.1 - t).sub(c.smul(Real.1 - t))
        triangle_smul_sub_distrib(b, c, Real.1 - t)
        b.sub(c).smul(Real.1 - t) = b.smul(Real.1 - t).sub(c.smul(Real.1 - t))
        d.sub(c) = b.sub(c).smul(Real.1 - t)
    }
}

/// The cross product of the bisector direction with the side direction
/// expands: `cross((1 - t) U + t V, v U + u V) = ((1 - t) u - t v) cross(U, V)`.
theorem triangle_angle_bisector_cross_expand(
    p: Point2[Real], q: Point2[Real], t: Real, u: Real, v: Real
) {
    p.smul(Real.1 - t).add(q.smul(t)).cross(q.smul(u).add(p.smul(v))) =
        ((Real.1 - t) * u - t * v) * p.cross(q)
} by {
    p.smul(Real.1 - t).add(q.smul(t)).cross(q.smul(u).add(p.smul(v))) =
        p.smul(Real.1 - t).cross(q.smul(u).add(p.smul(v))) +
        q.smul(t).cross(q.smul(u).add(p.smul(v)))
    point2_cross_add_left(p.smul(Real.1 - t), q.smul(t), q.smul(u).add(p.smul(v)))
    p.smul(Real.1 - t).cross(q.smul(u).add(p.smul(v))) =
        (Real.1 - t) * p.cross(q.smul(u).add(p.smul(v)))
    point2_cross_smul_left(Real.1 - t, p, q.smul(u).add(p.smul(v)))
    q.smul(t).cross(q.smul(u).add(p.smul(v))) = t * q.cross(q.smul(u).add(p.smul(v)))
    point2_cross_smul_left(t, q, q.smul(u).add(p.smul(v)))
    p.cross(q.smul(u).add(p.smul(v))) = p.cross(q.smul(u)) + p.cross(p.smul(v))
    point2_cross_add_right(p, q.smul(u), p.smul(v))
    p.cross(q.smul(u)) = u * p.cross(q)
    point2_cross_smul_right(u, p, q)
    p.cross(p.smul(v)) = v * p.cross(p)
    point2_cross_smul_right(v, p, p)
    point2_cross_self(p)
    p.cross(p) = Real.0
    v * p.cross(p) = Real.0
    p.cross(p.smul(v)) = Real.0
    p.cross(q.smul(u).add(p.smul(v))) = u * p.cross(q) + Real.0
    u * p.cross(q) + Real.0 = u * p.cross(q)
    p.cross(q.smul(u).add(p.smul(v))) = u * p.cross(q)
    q.cross(q.smul(u).add(p.smul(v))) = q.cross(q.smul(u)) + q.cross(p.smul(v))
    point2_cross_add_right(q, q.smul(u), p.smul(v))
    q.cross(q.smul(u)) = u * q.cross(q)
    point2_cross_smul_right(u, q, q)
    point2_cross_self(q)
    q.cross(q) = Real.0
    u * q.cross(q) = Real.0
    q.cross(q.smul(u)) = Real.0
    q.cross(p.smul(v)) = v * q.cross(p)
    point2_cross_smul_right(v, q, p)
    point2_cross_swap(q, p)
    q.cross(p) = -p.cross(q)
    v * q.cross(p) = -(v * p.cross(q))
    q.cross(p.smul(v)) = -(v * p.cross(q))
    q.cross(q.smul(u).add(p.smul(v))) = Real.0 + -(v * p.cross(q))
    Real.0 + -(v * p.cross(q)) = -(v * p.cross(q))
    q.cross(q.smul(u).add(p.smul(v))) = -(v * p.cross(q))
    p.smul(Real.1 - t).add(q.smul(t)).cross(q.smul(u).add(p.smul(v))) =
        (Real.1 - t) * (u * p.cross(q)) + t * (-(v * p.cross(q)))
    (Real.1 - t) * (u * p.cross(q)) = ((Real.1 - t) * u) * p.cross(q)
    t * (-(v * p.cross(q))) = -(t * (v * p.cross(q)))
    t * (v * p.cross(q)) = (t * v) * p.cross(q)
    -(t * (v * p.cross(q))) = -((t * v) * p.cross(q))
    ((Real.1 - t) * u) * p.cross(q) + -((t * v) * p.cross(q)) =
        ((Real.1 - t) * u) * p.cross(q) - (t * v) * p.cross(q)
    ((Real.1 - t) * u) * p.cross(q) - (t * v) * p.cross(q) =
        ((Real.1 - t) * u - t * v) * p.cross(q)
    p.smul(Real.1 - t).add(q.smul(t)).cross(q.smul(u).add(p.smul(v))) =
        ((Real.1 - t) * u - t * v) * p.cross(q)
}

/// A nonzero scalar times a point difference equals the point difference:
/// from `(1 - t) u = t v`, squaring gives `(1 - t)^2 u^2 = t^2 v^2`.
theorem triangle_angle_bisector_square_ratio(alpha: Real, u: Real, v: Real) {
    (Real.1 - alpha) * u = alpha * v
    implies
    (Real.1 - alpha) * (Real.1 - alpha) * (u * u) = alpha * alpha * (v * v)
} by {
    if (Real.1 - alpha) * u = alpha * v {
        ((Real.1 - alpha) * u) * ((Real.1 - alpha) * u) = (alpha * v) * (alpha * v)
        ((Real.1 - alpha) * u) * ((Real.1 - alpha) * u) =
            (Real.1 - alpha) * (Real.1 - alpha) * (u * u)
        (alpha * v) * (alpha * v) = alpha * alpha * (v * v)
        (Real.1 - alpha) * (Real.1 - alpha) * (u * u) = alpha * alpha * (v * v)
    }
}

/// The angle bisector theorem: if `d` lies on `bc` with parameter `t`,
/// `u^2 = |b - a|^2`, `v^2 = |c - a|^2`, the direction of `ad` is the
/// internal bisector direction `v (b - a) + u (c - a)`, and the triangle is
/// non-degenerate, then `|b - d|^2 v^2 = |d - c|^2 u^2`, i.e.
/// `(BD/DC)^2 = (AB/AC)^2`.
theorem triangle_angle_bisector(
    a: Point2[Real], b: Point2[Real], c: Point2[Real],
    d: Point2[Real], u: Real, v: Real, t: Real
) {
    u * u = b.sub(a).norm_sq and
    v * v = c.sub(a).norm_sq and
    b.smul(Real.1 - t).add(c.smul(t)) = d and
    d.sub(a).cross(b.sub(a).smul(v).add(c.sub(a).smul(u))) = Real.0 and
    b.sub(a).cross(c.sub(a)) != Real.0
    implies
    b.sub(d).norm_sq * (v * v) = d.sub(c).norm_sq * (u * u)
} by {
    if u * u = b.sub(a).norm_sq and
        v * v = c.sub(a).norm_sq and
        b.smul(Real.1 - t).add(c.smul(t)) = d and
        d.sub(a).cross(b.sub(a).smul(v).add(c.sub(a).smul(u))) = Real.0 and
        b.sub(a).cross(c.sub(a)) != Real.0 {
        u * u = b.sub(a).norm_sq
        v * v = c.sub(a).norm_sq
        b.smul(Real.1 - t).add(c.smul(t)) = d
        d.sub(a).cross(b.sub(a).smul(v).add(c.sub(a).smul(u))) = Real.0
        b.sub(a).cross(c.sub(a)) != Real.0
        // d - a = (1 - t)(b - a) + t (c - a)
        triangle_angle_bisector_d_sub_a(a, b, c, d, t)
        d.sub(a) = b.sub(a).smul(Real.1 - t).add(c.sub(a).smul(t))
        triangle_angle_bisector_cross_expand(b.sub(a), c.sub(a), t, u, v)
        b.sub(a).smul(Real.1 - t).add(c.sub(a).smul(t)).cross(
            c.sub(a).smul(u).add(b.sub(a).smul(v))) =
            ((Real.1 - t) * u - t * v) * b.sub(a).cross(c.sub(a))
        d.sub(a).cross(b.sub(a).smul(v).add(c.sub(a).smul(u))) =
            ((Real.1 - t) * u - t * v) * b.sub(a).cross(c.sub(a))
        ((Real.1 - t) * u - t * v) * b.sub(a).cross(c.sub(a)) = Real.0
        // cancel the nonzero cross product
        b.sub(a).cross(c.sub(a)) * ((Real.1 - t) * u - t * v) = Real.0
        triangle_mul_zero_cancel(
            b.sub(a).cross(c.sub(a)), (Real.1 - t) * u - t * v)
        (Real.1 - t) * u - t * v = Real.0
        (Real.1 - t) * u = t * v
        triangle_angle_bisector_square_ratio(t, u, v)
        (Real.1 - t) * (Real.1 - t) * (u * u) = t * t * (v * v)
        // |b - d|^2 = t^2 |c - b|^2
        triangle_angle_bisector_d_sub_b(b, c, d, t)
        d.sub(b) = c.sub(b).smul(t)
        point2_norm_sq_smul(t, c.sub(b))
        c.sub(b).smul(t).norm_sq = t * t * c.sub(b).norm_sq
        d.sub(b).norm_sq = t * t * c.sub(b).norm_sq
        point2_sub_reverse_neg(b, d)
        b.sub(d) = d.sub(b).neg
        point2_norm_sq_neg(d.sub(b))
        d.sub(b).neg.norm_sq = d.sub(b).norm_sq
        b.sub(d).norm_sq = d.sub(b).norm_sq
        point2_sub_reverse_neg(c, b)
        c.sub(b) = b.sub(c).neg
        point2_norm_sq_neg(b.sub(c))
        b.sub(c).neg.norm_sq = b.sub(c).norm_sq
        c.sub(b).norm_sq = b.sub(c).norm_sq
        b.sub(d).norm_sq = t * t * b.sub(c).norm_sq
        // |d - c|^2 = (1 - t)^2 |b - c|^2
        triangle_angle_bisector_d_sub_c(b, c, d, t)
        d.sub(c) = b.sub(c).smul(Real.1 - t)
        point2_norm_sq_smul(Real.1 - t, b.sub(c))
        b.sub(c).smul(Real.1 - t).norm_sq =
            (Real.1 - t) * (Real.1 - t) * b.sub(c).norm_sq
        d.sub(c).norm_sq = (Real.1 - t) * (Real.1 - t) * b.sub(c).norm_sq
        // the ratio identity
        b.sub(d).norm_sq * (v * v) =
            (t * t * b.sub(c).norm_sq) * (v * v)
        d.sub(c).norm_sq * (u * u) =
            ((Real.1 - t) * (Real.1 - t) * b.sub(c).norm_sq) * (u * u)
        (t * t * b.sub(c).norm_sq) * (v * v) =
            b.sub(c).norm_sq * (t * t * (v * v))
        ((Real.1 - t) * (Real.1 - t) * b.sub(c).norm_sq) * (u * u) =
            b.sub(c).norm_sq * ((Real.1 - t) * (Real.1 - t) * (u * u))
        (Real.1 - t) * (Real.1 - t) * (u * u) = t * t * (v * v)
        b.sub(c).norm_sq * ((Real.1 - t) * (Real.1 - t) * (u * u)) =
            b.sub(c).norm_sq * (t * t * (v * v))
        (t * t * b.sub(c).norm_sq) * (v * v) =
            ((Real.1 - t) * (Real.1 - t) * b.sub(c).norm_sq) * (u * u)
        b.sub(d).norm_sq * (v * v) = d.sub(c).norm_sq * (u * u)
    }
}

/// A circumcircle witness puts the three vertices on a common circle: with
/// `|a - o|^2 = |b - o|^2 = |c - o|^2 = radius_sq`, the vertices lie on the
/// circle centered at `o` with squared radius `radius_sq`.
theorem triangle_circumcenter_same_circle(
    o: Point2[Real], radius_sq: Real,
    a: Point2[Real], b: Point2[Real], c: Point2[Real]
) {
    a.sub(o).norm_sq = radius_sq and
    b.sub(o).norm_sq = radius_sq and
    c.sub(o).norm_sq = radius_sq
    implies
    o.same_circle(radius_sq, a, b) and o.same_circle(radius_sq, b, c) and
        o.same_circle(radius_sq, c, a)
} by {
    if a.sub(o).norm_sq = radius_sq and
        b.sub(o).norm_sq = radius_sq and
        c.sub(o).norm_sq = radius_sq {
        a.sub(o).norm_sq = radius_sq
        b.sub(o).norm_sq = radius_sq
        c.sub(o).norm_sq = radius_sq
        point2_dist_sq_eq_norm_sq_sub(a, o)
        a.dist_sq(o) = a.sub(o).norm_sq
        a.dist_sq(o) = radius_sq
        point2_center_dist_imp_on_circle(o, radius_sq, a)
        o.on_circle(radius_sq, a)
        point2_dist_sq_eq_norm_sq_sub(b, o)
        b.dist_sq(o) = b.sub(o).norm_sq
        b.dist_sq(o) = radius_sq
        point2_center_dist_imp_on_circle(o, radius_sq, b)
        o.on_circle(radius_sq, b)
        point2_dist_sq_eq_norm_sq_sub(c, o)
        c.dist_sq(o) = c.sub(o).norm_sq
        c.dist_sq(o) = radius_sq
        point2_center_dist_imp_on_circle(o, radius_sq, c)
        o.on_circle(radius_sq, c)
        o.on_circle(radius_sq, a) and o.on_circle(radius_sq, b)
        o.same_circle(radius_sq, a, b)
        o.on_circle(radius_sq, b) and o.on_circle(radius_sq, c)
        o.same_circle(radius_sq, b, c)
        o.on_circle(radius_sq, c) and o.on_circle(radius_sq, a)
        o.same_circle(radius_sq, c, a)
        o.same_circle(radius_sq, a, b) and o.same_circle(radius_sq, b, c) and
            o.same_circle(radius_sq, c, a)
    }
}
