from ordered_field import OrderedField
from list import List
from geometry.point2 import Point2
from geometry.point2_affine import point2_translate_neg_right
from geometry.point2_polygon import point2_translate_points
from geometry.point2_polygon_edges import point2_polyline_chain_contains,
    point2_polyline_chain_contains_translate_forward, point2_polyline_contains,
    point2_polyline_contains_pair
from geometry.point2_polyline_translate_bridge import point2_translate_points_neg_right
from geometry.point2_segment import point2_on_segment_swap

/// If a translated point lies in the translated edge chain, the original point lies in the original chain.
theorem point2_polyline_chain_contains_translate_backward[T: OrderedField](
    previous: Point2[T], rest: List[Point2[T]], p: Point2[T], v: Point2[T]
) {
    point2_polyline_chain_contains(previous.translate(v), point2_translate_points(rest, v), p.translate(v))
    implies point2_polyline_chain_contains(previous, rest, p)
} by {
    if point2_polyline_chain_contains(previous.translate(v), point2_translate_points(rest, v), p.translate(v)) {
        point2_polyline_chain_contains_translate_forward(
            previous.translate(v), point2_translate_points(rest, v), p.translate(v), v.neg
        )
        point2_polyline_chain_contains(
            previous.translate(v).translate(v.neg),
            point2_translate_points(point2_translate_points(rest, v), v.neg),
            p.translate(v).translate(v.neg)
        )
        point2_translate_neg_right(previous, v)
        point2_translate_points_neg_right(rest, v)
        point2_translate_neg_right(p, v)
        point2_polyline_chain_contains(previous, rest, p)
    }
}

/// Translating an edge chain and its query point preserves chain membership exactly.
theorem point2_polyline_chain_contains_translate_iff[T: OrderedField](
    previous: Point2[T], rest: List[Point2[T]], p: Point2[T], v: Point2[T]
) {
    point2_polyline_chain_contains(previous.translate(v), point2_translate_points(rest, v), p.translate(v)) =
    point2_polyline_chain_contains(previous, rest, p)
} by {
    if point2_polyline_chain_contains(previous, rest, p) {
        point2_polyline_chain_contains_translate_forward(previous, rest, p, v)
        point2_polyline_chain_contains(previous.translate(v), point2_translate_points(rest, v), p.translate(v))
    }
    if point2_polyline_chain_contains(previous.translate(v), point2_translate_points(rest, v), p.translate(v)) {
        point2_polyline_chain_contains_translate_backward(previous, rest, p, v)
        point2_polyline_chain_contains(previous, rest, p)
    }
}

/// Reversing the two endpoints of a two-point polyline preserves membership.
theorem point2_polyline_contains_pair_swap[T: OrderedField](a: Point2[T], b: Point2[T], p: Point2[T]) {
    point2_polyline_contains(List.cons(a, List.cons(b, List.nil[Point2[T]])), p) =
    point2_polyline_contains(List.cons(b, List.cons(a, List.nil[Point2[T]])), p)
} by {
    point2_polyline_contains_pair(a, b, p)
    point2_polyline_contains_pair(b, a, p)
    point2_on_segment_swap(a, b, p)
}
