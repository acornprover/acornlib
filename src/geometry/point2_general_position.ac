from nat import Nat
from comm_ring import CommRing
from ordered_field import OrderedField
from finite_set import FiniteSet, fs_image, finite_set_image_contains_eq,
    finite_set_subset_contains
from data.finite.finite_set_card import fs_card
from data.finite.finite_set_membership import finite_set_eq_of_contains_eq
from geometry.point2 import Point2
from geometry.point2_orientation import point2_collinear_rotate,
    point2_collinear_swap_first_second, point2_collinear_swap_second_third
from data.finite.finite_set_card import fs_card_singleton
from data.finite.finite_set_card_compare import fs_card_mono
from data.finite.finite_set_subset_intro import fs_subset_eq_intro
from finite_set import finite_set_singleton_contains_eq
from geometry.point2_concyclic import concyclic4, concyclic4_of_equal_distances
from geometry.point2_distance_set import distances_from, distances_from_contains,
    distances_from_witness, distance_count, dist_sq_from, no_four_concyclic,
    no_four_concyclic_apply, four_distinct_in

numerals Nat

/// True when three points are distinct members of the set.
///
/// The three-point counterpart of `four_distinct_in`, named for the same reason: it keeps the
/// general-position condition below a two-part statement.
define three_distinct_in[T: OrderedField](
    s: FiniteSet[Point2[T]], a: Point2[T], b: Point2[T], c: Point2[T]
) -> Bool {
    s.contains(a) and s.contains(b) and s.contains(c)
        and a != b and a != c and b != c
}

/// The three points lie in the set and are pairwise different.
theorem three_distinct_in_apply[T: OrderedField](
    s: FiniteSet[Point2[T]], a: Point2[T], b: Point2[T], c: Point2[T]
) {
    three_distinct_in(s, a, b, c)
        implies s.contains(a) and s.contains(b) and s.contains(c)
            and a != b and a != c and b != c
} by {
    if three_distinct_in(s, a, b, c) {
        three_distinct_in(s, a, b, c) =
            (s.contains(a) and s.contains(b) and s.contains(c)
                and a != b and a != c and b != c)
        (s.contains(a) and s.contains(b) and s.contains(c)
            and a != b and a != c and b != c)
    }
}

/// Three distinct members give the condition.
theorem three_distinct_in_intro[T: OrderedField](
    s: FiniteSet[Point2[T]], a: Point2[T], b: Point2[T], c: Point2[T]
) {
    s.contains(a) and s.contains(b) and s.contains(c)
        and a != b and a != c and b != c
        implies three_distinct_in(s, a, b, c)
} by {
    if s.contains(a) and s.contains(b) and s.contains(c)
        and a != b and a != c and b != c {
        three_distinct_in(s, a, b, c) =
            (s.contains(a) and s.contains(b) and s.contains(c)
                and a != b and a != c and b != c)
        three_distinct_in(s, a, b, c)
    }
}

/// Three distinct members of a subset are three distinct members of the whole.
theorem three_distinct_in_of_subset[T: OrderedField](
    s: FiniteSet[Point2[T]], u: FiniteSet[Point2[T]],
    a: Point2[T], b: Point2[T], c: Point2[T]
) {
    s.subset_eq(u) and three_distinct_in(s, a, b, c) implies three_distinct_in(u, a, b, c)
} by {
    if s.subset_eq(u) and three_distinct_in(s, a, b, c) {
        three_distinct_in_apply(s, a, b, c)
        s.contains(a)
        finite_set_subset_contains(s, u, a)
        u.contains(a)
        s.contains(b)
        finite_set_subset_contains(s, u, b)
        u.contains(b)
        s.contains(c)
        finite_set_subset_contains(s, u, c)
        u.contains(c)
        a != b
        a != c
        b != c
        three_distinct_in_intro(u, a, b, c)
        three_distinct_in(u, a, b, c)
    }
}

/// True when no three distinct points of the set are collinear.
///
/// General position in the usual sense. Stated through the existing `collinear` predicate,
/// which is the vanishing of the orientation determinant, so the orientation lemmas apply
/// directly to it.
define in_general_position[T: OrderedField](s: FiniteSet[Point2[T]]) -> Bool {
    forall(a: Point2[T], b: Point2[T], c: Point2[T]) {
        (three_distinct_in(s, a, b, c) implies not a.collinear(b, c))
    }
}

/// Three distinct points of such a set are not collinear.
theorem in_general_position_apply[T: OrderedField](
    s: FiniteSet[Point2[T]], a: Point2[T], b: Point2[T], c: Point2[T]
) {
    in_general_position(s) and three_distinct_in(s, a, b, c)
        implies not a.collinear(b, c)
} by {
    if in_general_position(s) and three_distinct_in(s, a, b, c) {
        in_general_position(s) = forall(x: Point2[T], y: Point2[T], z: Point2[T]) {
            (three_distinct_in(s, x, y, z) implies not x.collinear(y, z))
        }
        forall(x: Point2[T], y: Point2[T], z: Point2[T]) {
            (three_distinct_in(s, x, y, z) implies not x.collinear(y, z))
        }
        (three_distinct_in(s, a, b, c) implies not a.collinear(b, c))
        not a.collinear(b, c)
    }
}

/// The pointwise condition is general position.
theorem in_general_position_intro[T: OrderedField](s: FiniteSet[Point2[T]]) {
    (forall(a: Point2[T], b: Point2[T], c: Point2[T]) {
        (three_distinct_in(s, a, b, c) implies not a.collinear(b, c))
    }) implies in_general_position(s)
} by {
    if forall(a: Point2[T], b: Point2[T], c: Point2[T]) {
        (three_distinct_in(s, a, b, c) implies not a.collinear(b, c))
    } {
        in_general_position(s) = forall(x: Point2[T], y: Point2[T], z: Point2[T]) {
            (three_distinct_in(s, x, y, z) implies not x.collinear(y, z))
        }
        in_general_position(s)
    }
}

/// General position passes to subsets.
theorem in_general_position_of_subset[T: OrderedField](
    s: FiniteSet[Point2[T]], u: FiniteSet[Point2[T]]
) {
    s.subset_eq(u) and in_general_position(u) implies in_general_position(s)
} by {
    if s.subset_eq(u) and in_general_position(u) {
        forall(a: Point2[T], b: Point2[T], c: Point2[T]) {
            if three_distinct_in(s, a, b, c) {
                three_distinct_in_of_subset(s, u, a, b, c)
                three_distinct_in(u, a, b, c)
                in_general_position_apply(u, a, b, c)
                not a.collinear(b, c)
            }
            (three_distinct_in(s, a, b, c) implies not a.collinear(b, c))
        }
        in_general_position_intro(s)
        in_general_position(s)
    }
}

/// Non-collinearity of three points does not depend on their order.
///
/// Collinearity is invariant under rearranging its arguments, so its negation is too, and the
/// general-position condition can be applied to the three points in whatever order is at hand.
theorem in_general_position_any_order[T: OrderedField](
    s: FiniteSet[Point2[T]], a: Point2[T], b: Point2[T], c: Point2[T]
) {
    in_general_position(s) and three_distinct_in(s, a, b, c)
        implies not b.collinear(c, a) and not b.collinear(a, c)
} by {
    if in_general_position(s) and three_distinct_in(s, a, b, c) {
        in_general_position_apply(s, a, b, c)
        not a.collinear(b, c)
        if b.collinear(c, a) {
            point2_collinear_rotate(b, c, a)
            a.collinear(b, c)
            false
        }
        not b.collinear(c, a)
        if b.collinear(a, c) {
            point2_collinear_swap_first_second(b, a, c)
            a.collinear(b, c)
            false
        }
        not b.collinear(a, c)
        not b.collinear(c, a) and not b.collinear(a, c)
    }
}

/// True when a planar map preserves squared distances.
///
/// The right notion here: an isometry in the metric sense preserves distances, and squaring
/// is injective on nonnegative values, so preserving squared distances is the same condition
/// and needs no square roots.
define preserves_dist_sq[T: OrderedField](f: Point2[T] -> Point2[T]) -> Bool {
    forall(p: Point2[T], q: Point2[T]) {
        f(p).dist_sq(f(q)) = p.dist_sq(q)
    }
}

/// Such a map preserves the squared distance between any two points.
theorem preserves_dist_sq_apply[T: OrderedField](
    f: Point2[T] -> Point2[T], p: Point2[T], q: Point2[T]
) {
    preserves_dist_sq(f) implies f(p).dist_sq(f(q)) = p.dist_sq(q)
} by {
    if preserves_dist_sq(f) {
        preserves_dist_sq(f) = forall(x: Point2[T], y: Point2[T]) {
            f(x).dist_sq(f(y)) = x.dist_sq(y)
        }
        forall(x: Point2[T], y: Point2[T]) {
            f(x).dist_sq(f(y)) = x.dist_sq(y)
        }
        f(p).dist_sq(f(q)) = p.dist_sq(q)
    }
}

/// The pointwise condition gives a distance-preserving map.
theorem preserves_dist_sq_intro[T: OrderedField](f: Point2[T] -> Point2[T]) {
    (forall(p: Point2[T], q: Point2[T]) { f(p).dist_sq(f(q)) = p.dist_sq(q) })
        implies preserves_dist_sq(f)
} by {
    if forall(p: Point2[T], q: Point2[T]) { f(p).dist_sq(f(q)) = p.dist_sq(q) } {
        preserves_dist_sq(f) = forall(x: Point2[T], y: Point2[T]) {
            f(x).dist_sq(f(y)) = x.dist_sq(y)
        }
        preserves_dist_sq(f)
    }
}

/// A distance-preserving map carries the distance set onto the distance set of the image.
///
/// Each realized distance is realized by the image of the point that realized it, and every
/// distance realized in the image comes from one realized in the original.
theorem distances_from_image_eq[T: OrderedField](
    f: Point2[T] -> Point2[T], p: Point2[T], s: FiniteSet[Point2[T]]
) {
    preserves_dist_sq(f)
        implies distances_from(f(p), fs_image(s, f)) = distances_from(p, s)
} by {
    if preserves_dist_sq(f) {
        forall(t: T) {
            if distances_from(f(p), fs_image(s, f)).contains(t) {
                distances_from_witness(f(p), fs_image(s, f), t)
                let (r: Point2[T]) satisfy {
                    fs_image(s, f).contains(r) and t = r.dist_sq(f(p))
                }
                finite_set_image_contains_eq(s, f, r)
                exists(q: Point2[T]) {
                    s.contains(q) and r = f(q)
                }
                let (q: Point2[T]) satisfy {
                    s.contains(q) and r = f(q)
                }
                t = f(q).dist_sq(f(p))
                preserves_dist_sq_apply(f, q, p)
                f(q).dist_sq(f(p)) = q.dist_sq(p)
                t = q.dist_sq(p)
                distances_from_contains(p, s, q)
                distances_from(p, s).contains(q.dist_sq(p))
                distances_from(p, s).contains(t)
            }
            if distances_from(p, s).contains(t) {
                distances_from_witness(p, s, t)
                let (q: Point2[T]) satisfy {
                    s.contains(q) and t = q.dist_sq(p)
                }
                finite_set_image_contains_eq(s, f, f(q))
                exists(r: Point2[T]) {
                    s.contains(r) and f(q) = f(r)
                }
                fs_image(s, f).contains(f(q))
                preserves_dist_sq_apply(f, q, p)
                f(q).dist_sq(f(p)) = q.dist_sq(p)
                distances_from_contains(f(p), fs_image(s, f), f(q))
                distances_from(f(p), fs_image(s, f)).contains(f(q).dist_sq(f(p)))
                distances_from(f(p), fs_image(s, f)).contains(t)
            }
            (distances_from(f(p), fs_image(s, f)).contains(t)
                implies distances_from(p, s).contains(t))
            (distances_from(p, s).contains(t)
                implies distances_from(f(p), fs_image(s, f)).contains(t))
            distances_from(f(p), fs_image(s, f)).contains(t) = distances_from(p, s).contains(t)
        }
        finite_set_eq_of_contains_eq(distances_from(f(p), fs_image(s, f)), distances_from(p, s))
        distances_from(f(p), fs_image(s, f)) = distances_from(p, s)
    }
}

/// Distance counts are invariant under a distance-preserving map.
///
/// The two distance sets are equal, so their cardinalities are.
theorem distance_count_isometry_invariant[T: OrderedField](
    f: Point2[T] -> Point2[T], p: Point2[T], s: FiniteSet[Point2[T]]
) {
    preserves_dist_sq(f)
        implies distance_count(f(p), fs_image(s, f)) = distance_count(p, s)
} by {
    if preserves_dist_sq(f) {
        distances_from_image_eq(f, p, s)
        distances_from(f(p), fs_image(s, f)) = distances_from(p, s)
        fs_card(distances_from(f(p), fs_image(s, f))) = fs_card(distances_from(p, s))
        distance_count(f(p), fs_image(s, f)) = distance_count(p, s)
    }
}

/// Four points equidistant from a common centre are concyclic.
///
/// Being equidistant from a point *is* lying on a circle about it, so this is the bridge
/// between the metric condition and the incidence one. It is what gives the no-four-concyclic
/// condition its meaning for distance counting.
theorem equidistant_four_are_concyclic[T: OrderedField](
    centre: Point2[T], a: Point2[T], b: Point2[T], c: Point2[T], d: Point2[T]
) {
    a.dist_sq(centre) = b.dist_sq(centre) and a.dist_sq(centre) = c.dist_sq(centre)
        and a.dist_sq(centre) = d.dist_sq(centre)
        implies concyclic4(a, b, c, d)
} by {
    if a.dist_sq(centre) = b.dist_sq(centre) and a.dist_sq(centre) = c.dist_sq(centre)
        and a.dist_sq(centre) = d.dist_sq(centre) {
        concyclic4_of_equal_distances(centre, a, b, c, d)
        concyclic4(a, b, c, d)
    }
}

/// In a set with no four concyclic points, no four distinct members are equidistant from any
/// point.
///
/// The contrapositive of the previous theorem. Since points equidistant from `p` are exactly
/// the points of `s` contributing one particular value to `distances_from(p, s)`, this says
/// each realised distance is attained by at most three points — the statement that makes
/// no-four-concyclic a hypothesis about distance counting rather than about circles.
theorem no_four_equidistant[T: OrderedField](
    s: FiniteSet[Point2[T]], p: Point2[T],
    a: Point2[T], b: Point2[T], c: Point2[T], d: Point2[T]
) {
    no_four_concyclic(s) and four_distinct_in(s, a, b, c, d)
        implies not (a.dist_sq(p) = b.dist_sq(p) and a.dist_sq(p) = c.dist_sq(p)
            and a.dist_sq(p) = d.dist_sq(p))
} by {
    if no_four_concyclic(s) and four_distinct_in(s, a, b, c, d) {
        if a.dist_sq(p) = b.dist_sq(p) and a.dist_sq(p) = c.dist_sq(p)
            and a.dist_sq(p) = d.dist_sq(p) {
            equidistant_four_are_concyclic(p, a, b, c, d)
            concyclic4(a, b, c, d)
            no_four_concyclic_apply(s, a, b, c, d)
            not concyclic4(a, b, c, d)
            false
        }
        not (a.dist_sq(p) = b.dist_sq(p) and a.dist_sq(p) = c.dist_sq(p)
            and a.dist_sq(p) = d.dist_sq(p))
    }
}

/// A nonempty point set realizes at least one distance.
theorem distance_count_positive[T: OrderedField](
    p: Point2[T], s: FiniteSet[Point2[T]], q: Point2[T]
) {
    s.contains(q) implies Nat.1 <= distance_count(p, s)
} by {
    if s.contains(q) {
        distances_from_contains(p, s, q)
        distances_from(p, s).contains(q.dist_sq(p))
        forall(t: T) {
            if FiniteSet.empty[T].insert(q.dist_sq(p)).contains(t) {
                finite_set_singleton_contains_eq(q.dist_sq(p), t)
                t = q.dist_sq(p)
                distances_from(p, s).contains(t)
            }
            (FiniteSet.empty[T].insert(q.dist_sq(p)).contains(t)
                implies distances_from(p, s).contains(t))
        }
        fs_subset_eq_intro(FiniteSet.empty[T].insert(q.dist_sq(p)), distances_from(p, s))
        FiniteSet.empty[T].insert(q.dist_sq(p)).subset_eq(distances_from(p, s))
        fs_card_mono(FiniteSet.empty[T].insert(q.dist_sq(p)), distances_from(p, s))
        (fs_card(FiniteSet.empty[T].insert(q.dist_sq(p)))
            <= fs_card(distances_from(p, s)))
        fs_card_singleton(q.dist_sq(p))
        fs_card(FiniteSet.empty[T].insert(q.dist_sq(p))) = Nat.1
        Nat.1 <= fs_card(distances_from(p, s))
        Nat.1 <= distance_count(p, s)
    }
}
