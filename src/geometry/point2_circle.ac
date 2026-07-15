from ordered_field import OrderedField
from geometry.point2 import Point2
from geometry.point2_algebra import point2_dist_sq_self, point2_dist_sq_comm
from geometry.point2_metric import point2_dist_sq_nonneg,
    point2_dist_sq_translate_center, point2_dist_sq_center_symm

attributes Point2[T: OrderedField] {
    /// True when `p` lies on the circle with this center and squared radius.
    define on_circle(self, radius_sq: T, p: Point2[T]) -> Bool {
        p.dist_sq(self) = radius_sq
    }

    /// True when two points lie on the same circle with this center and squared radius.
    define same_circle(self, radius_sq: T, p: Point2[T], q: Point2[T]) -> Bool {
        self.on_circle(radius_sq, p) and self.on_circle(radius_sq, q)
    }
}

/// Circle membership is equality of squared distance to the center.
theorem point2_on_circle_eq_dist_sq[T: OrderedField](center: Point2[T], radius_sq: T, p: Point2[T]) {
    center.on_circle(radius_sq, p) = (p.dist_sq(center) = radius_sq)
}

/// The center lies on the circle of squared radius zero.
theorem point2_center_on_zero_circle[T: OrderedField](center: Point2[T]) {
    center.on_circle(T.0, center)
} by {
    point2_dist_sq_self(center)
}

/// A point on a circle determines a nonnegative squared radius.
theorem point2_on_circle_radius_sq_nonneg[T: OrderedField](center: Point2[T], radius_sq: T, p: Point2[T]) {
    center.on_circle(radius_sq, p) implies radius_sq >= T.0
} by {
    if center.on_circle(radius_sq, p) {
        p.dist_sq(center) = radius_sq
        point2_dist_sq_nonneg(p, center)
    }
}

/// Circle membership can use center-to-point squared distance.
theorem point2_on_circle_center_dist[T: OrderedField](center: Point2[T], radius_sq: T, p: Point2[T]) {
    center.on_circle(radius_sq, p) implies center.dist_sq(p) = radius_sq
} by {
    if center.on_circle(radius_sq, p) {
        p.dist_sq(center) = radius_sq
        point2_dist_sq_center_symm(p, center)
    }
}

/// Center-to-point squared distance gives circle membership.
theorem point2_center_dist_imp_on_circle[T: OrderedField](center: Point2[T], radius_sq: T, p: Point2[T]) {
    center.dist_sq(p) = radius_sq implies center.on_circle(radius_sq, p)
} by {
    if center.dist_sq(p) = radius_sq {
        point2_dist_sq_comm(center, p)
        p.dist_sq(center) = radius_sq
    }
}

/// Translating center and point preserves circle membership.
theorem point2_on_circle_translate[T: OrderedField](center: Point2[T], radius_sq: T, p: Point2[T], v: Point2[T]) {
    center.translate(v).on_circle(radius_sq, p.translate(v)) = center.on_circle(radius_sq, p)
} by {
    point2_dist_sq_translate_center(p, center, v)
}

/// Translating a point on a circle gives a point on the translated circle.
theorem point2_on_circle_translate_forward[T: OrderedField](center: Point2[T], radius_sq: T, p: Point2[T], v: Point2[T]) {
    center.on_circle(radius_sq, p) implies center.translate(v).on_circle(radius_sq, p.translate(v))
} by {
    if center.on_circle(radius_sq, p) {
        point2_on_circle_translate(center, radius_sq, p, v)
    }
}

/// Translating a zero-radius circle gives a zero-radius circle.
theorem point2_center_on_zero_circle_translate[T: OrderedField](center: Point2[T], v: Point2[T]) {
    center.translate(v).on_circle(T.0, center.translate(v))
} by {
    point2_center_on_zero_circle(center.translate(v))
}

/// Two points on the same circle have equal squared distance from the center.
theorem point2_same_circle_dist_sq_eq[T: OrderedField](center: Point2[T], radius_sq: T, p: Point2[T], q: Point2[T]) {
    center.same_circle(radius_sq, p, q) implies p.dist_sq(center) = q.dist_sq(center)
} by {
    if center.same_circle(radius_sq, p, q) {
        center.on_circle(radius_sq, p)
        center.on_circle(radius_sq, q)
        q.dist_sq(center) = radius_sq
    }
}

/// Same-circle membership is symmetric in the two points.
theorem point2_same_circle_comm[T: OrderedField](center: Point2[T], radius_sq: T, p: Point2[T], q: Point2[T]) {
    center.same_circle(radius_sq, p, q) = center.same_circle(radius_sq, q, p)
} by {
    if center.same_circle(radius_sq, p, q) {
        center.on_circle(radius_sq, q)
        center.same_circle(radius_sq, q, p)
    }
    if center.same_circle(radius_sq, q, p) {
        center.on_circle(radius_sq, p)
        center.same_circle(radius_sq, p, q)
    }
}

/// A point on a circle is on the same circle as itself.
theorem point2_same_circle_self[T: OrderedField](center: Point2[T], radius_sq: T, p: Point2[T]) {
    center.on_circle(radius_sq, p) implies center.same_circle(radius_sq, p, p)
}

/// Translating a same-circle pair preserves same-circle membership.
theorem point2_same_circle_translate[T: OrderedField](center: Point2[T], radius_sq: T, p: Point2[T], q: Point2[T], v: Point2[T]) {
    center.translate(v).same_circle(radius_sq, p.translate(v), q.translate(v)) =
    center.same_circle(radius_sq, p, q)
} by {
    point2_on_circle_translate(center, radius_sq, p, v)
    point2_on_circle_translate(center, radius_sq, q, v)
}

/// Translating a same-circle pair gives a same-circle pair.
theorem point2_same_circle_translate_forward[T: OrderedField](center: Point2[T], radius_sq: T, p: Point2[T], q: Point2[T], v: Point2[T]) {
    center.same_circle(radius_sq, p, q) implies
    center.translate(v).same_circle(radius_sq, p.translate(v), q.translate(v))
} by {
    if center.same_circle(radius_sq, p, q) {
        point2_same_circle_translate(center, radius_sq, p, q, v)
    }
}

/// A point lies on the zero circle centered at itself.
theorem point2_on_circle_self_zero_radius[T: OrderedField](p: Point2[T]) {
    p.on_circle(T.0, p)
} by {
    point2_center_on_zero_circle(p)
}

/// A same-circle pair determines a nonnegative squared radius.
theorem point2_same_circle_radius_sq_nonneg[T: OrderedField](center: Point2[T], radius_sq: T, p: Point2[T], q: Point2[T]) {
    center.same_circle(radius_sq, p, q) implies radius_sq >= T.0
} by {
    if center.same_circle(radius_sq, p, q) {
        center.on_circle(radius_sq, p)
        point2_on_circle_radius_sq_nonneg(center, radius_sq, p)
    }
}

/// Same-circle membership gives center-to-point equality for the first point.
theorem point2_same_circle_first_center_dist[T: OrderedField](center: Point2[T], radius_sq: T, p: Point2[T], q: Point2[T]) {
    center.same_circle(radius_sq, p, q) implies center.dist_sq(p) = radius_sq
} by {
    if center.same_circle(radius_sq, p, q) {
        center.on_circle(radius_sq, p)
        point2_on_circle_center_dist(center, radius_sq, p)
    }
}

/// Same-circle membership gives center-to-point equality for the second point.
theorem point2_same_circle_second_center_dist[T: OrderedField](center: Point2[T], radius_sq: T, p: Point2[T], q: Point2[T]) {
    center.same_circle(radius_sq, p, q) implies center.dist_sq(q) = radius_sq
} by {
    if center.same_circle(radius_sq, p, q) {
        center.on_circle(radius_sq, q)
        point2_on_circle_center_dist(center, radius_sq, q)
    }
}

/// If a circle contains a point, the point-center distance equals the squared radius in either order.
theorem point2_on_circle_dist_sq_symmetric[T: OrderedField](center: Point2[T], radius_sq: T, p: Point2[T]) {
    center.on_circle(radius_sq, p) implies
    p.dist_sq(center) = radius_sq and center.dist_sq(p) = radius_sq
} by {
    if center.on_circle(radius_sq, p) {
        p.dist_sq(center) = radius_sq
        point2_on_circle_center_dist(center, radius_sq, p)
    }
}

/// A translated point on a circle determines a nonnegative squared radius.
theorem point2_on_circle_translate_radius_sq_nonneg[T: OrderedField](center: Point2[T], radius_sq: T, p: Point2[T], v: Point2[T]) {
    center.translate(v).on_circle(radius_sq, p.translate(v)) implies radius_sq >= T.0
} by {
    if center.translate(v).on_circle(radius_sq, p.translate(v)) {
        point2_on_circle_radius_sq_nonneg(center.translate(v), radius_sq, p.translate(v))
    }
}

/// A translated same-circle pair determines a nonnegative squared radius.
theorem point2_same_circle_translate_radius_sq_nonneg[T: OrderedField](center: Point2[T], radius_sq: T, p: Point2[T], q: Point2[T], v: Point2[T]) {
    center.translate(v).same_circle(radius_sq, p.translate(v), q.translate(v)) implies radius_sq >= T.0
} by {
    if center.translate(v).same_circle(radius_sq, p.translate(v), q.translate(v)) {
        point2_same_circle_radius_sq_nonneg(center.translate(v), radius_sq, p.translate(v), q.translate(v))
    }
}
