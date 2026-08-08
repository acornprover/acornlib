from algebra.add_comm_group import AddCommGroup
from comm_ring import CommRing
from ordered_field import OrderedField
from geometry.point2 import Point2, point2_zero, point2_ext,
    point2_add_comm, point2_add_zero_right, point2_add_neg_right,
    point2_sub_self
from geometry.point2_algebra import point2_sub_add_sub,
    point2_scalar_sub_reverse_neg, point2_cross_self, point2_cross_smul_right

/// Scalar multiplication by zero gives the origin.
theorem point2_smul_zero_left[T: CommRing](p: Point2[T]) {
    p.smul(T.0) = point2_zero[T]
} by {
    let lhs = p.smul(T.0)
    point2_ext(lhs, point2_zero[T])
}

/// Scalar multiplication by one leaves a point unchanged.
theorem point2_smul_one_left[T: CommRing](p: Point2[T]) {
    p.smul(T.1) = p
} by {
    let lhs = p.smul(T.1)
    point2_ext(lhs, p)
}

/// Adding a displacement from `x` to `y` to `x` gives `y`.
theorem point2_scalar_add_sub_left_cancel[T: AddCommGroup](x: T, y: T) {
    x + (y - x) = y
} by {
    (x + -x) + y = T.0 + y
}

/// Adding the displacement from one point to another gives the other point.
theorem point2_add_sub_left_cancel[T: AddCommGroup](a: Point2[T], b: Point2[T]) {
    a.add(b.sub(a)) = b
} by {
    let lhs = a.add(b.sub(a))
    point2_scalar_add_sub_left_cancel(a.x, b.x)
    lhs.x = b.x
    point2_scalar_add_sub_left_cancel(a.y, b.y)
    lhs.y = b.y
    point2_ext(lhs, b)
}

/// Reversing a parameter on a directed line gives the same coordinate.
theorem point2_scalar_param_line_reverse[T: CommRing](x: T, y: T, t: T) {
    y + (T.1 - t) * (x - y) = x + t * (y - x)
} by {
    let d = x - y
    point2_scalar_add_sub_left_cancel(y, x)
    point2_scalar_sub_reverse_neg(y, x)
    y + (d + -(t * d)) = y + d + -(t * d)
}

/// The last two summands in a triple point sum may be exchanged.
theorem point2_add_right_comm[T: AddCommGroup](a: Point2[T], b: Point2[T], c: Point2[T]) {
    a.add(b).add(c) = a.add(c).add(b)
} by {
    point2_add_comm(b, c)
    let lhs = a.add(b).add(c)
    let rhs = a.add(c).add(b)
    lhs.x = a.x + b.x + c.x
    rhs.x = a.x + c.x + b.x
    b.x + c.x = c.x + b.x
    lhs.x = rhs.x
    lhs.y = a.y + b.y + c.y
    rhs.y = a.y + c.y + b.y
    b.y + c.y = c.y + b.y
    point2_ext(lhs, rhs)
}

/// Translating by the origin leaves a point unchanged.
theorem point2_translate_zero_right[T: AddCommGroup](p: Point2[T]) {
    p.translate(point2_zero[T]) = p
} by {
    point2_add_zero_right(p)
}

/// Successive translations compose by point addition.
theorem point2_translate_translate[T: AddCommGroup](p: Point2[T], u: Point2[T], v: Point2[T]) {
    p.translate(u).translate(v) = p.translate(u.add(v))
} by {
    let lhs = p.translate(u).translate(v)
    let rhs = p.translate(u.add(v))
    lhs.x = p.x + u.x + v.x
    rhs.x = p.x + (u.x + v.x)
    lhs.x = rhs.x
    lhs.y = p.y + u.y + v.y
    rhs.y = p.y + (u.y + v.y)
    lhs.y = rhs.y
    point2_ext(lhs, rhs)
}

/// Translating by a point and then by its negative gives the original point.
theorem point2_translate_neg_right[T: AddCommGroup](p: Point2[T], v: Point2[T]) {
    p.translate(v).translate(v.neg) = p
} by {
    point2_translate_translate(p, v, v.neg)
    point2_add_neg_right(v)
    point2_translate_zero_right(p)
}

/// Translating two points by the same point reflects equality.
theorem point2_translate_cancel[T: AddCommGroup](p: Point2[T], q: Point2[T], v: Point2[T]) {
    p.translate(v) = q.translate(v) implies p = q
} by {
    if p.translate(v) = q.translate(v) {
        point2_translate_neg_right(p, v)
        point2_translate_neg_right(q, v)
        p = q
    }
}

/// Translating both endpoints of a displacement by the same point preserves the displacement.
theorem point2_sub_translate[T: AddCommGroup](a: Point2[T], b: Point2[T], v: Point2[T]) {
    a.translate(v).sub(b.translate(v)) = a.sub(b)
} by {
    point2_sub_add_sub(a, v, b, v)
    point2_sub_self(v)
    point2_add_zero_right(a.sub(b))
}

/// Translating both points preserves squared distance.
theorem point2_dist_sq_translate[T: CommRing](a: Point2[T], b: Point2[T], v: Point2[T]) {
    a.translate(v).dist_sq(b.translate(v)) = a.dist_sq(b)
} by {
    point2_sub_translate(a, b, v)
}

/// Translating all three points preserves orientation.
theorem point2_orientation_translate[T: CommRing](a: Point2[T], b: Point2[T], c: Point2[T], v: Point2[T]) {
    a.translate(v).orientation(b.translate(v), c.translate(v)) = a.orientation(b, c)
} by {
    point2_sub_translate(b, a, v)
    point2_sub_translate(c, a, v)
}

/// Translating all three points preserves collinearity.
theorem point2_collinear_translate[T: CommRing](a: Point2[T], b: Point2[T], c: Point2[T], v: Point2[T]) {
    a.translate(v).collinear(b.translate(v), c.translate(v)) = a.collinear(b, c)
} by {
    point2_orientation_translate(a, b, c, v)
}

/// Translating all three points preserves left turns.
theorem point2_left_turn_translate[T: OrderedField](a: Point2[T], b: Point2[T], c: Point2[T], v: Point2[T]) {
    a.translate(v).left_turn(b.translate(v), c.translate(v)) = a.left_turn(b, c)
} by {
    point2_orientation_translate(a, b, c, v)
}

/// Translating all three points preserves right turns.
theorem point2_right_turn_translate[T: OrderedField](a: Point2[T], b: Point2[T], c: Point2[T], v: Point2[T]) {
    a.translate(v).right_turn(b.translate(v), c.translate(v)) = a.right_turn(b, c)
} by {
    point2_orientation_translate(a, b, c, v)
}

/// The zero parameter on the directed line from `a` to `b` is `a`.
theorem point2_param_line_zero[T: CommRing](a: Point2[T], b: Point2[T]) {
    a.param_line(b, T.0) = a
} by {
    point2_smul_zero_left(b.sub(a))
    point2_add_zero_right(a)
}

/// The unit parameter on the directed line from `a` to `b` is `b`.
theorem point2_param_line_one[T: CommRing](a: Point2[T], b: Point2[T]) {
    a.param_line(b, T.1) = b
} by {
    point2_smul_one_left(b.sub(a))
    let lhs = a.add(b.sub(a))
    point2_scalar_add_sub_left_cancel(a.x, b.x)
    lhs.x = b.x
    lhs.y = a.y + (b.y - a.y)
    point2_scalar_add_sub_left_cancel(a.y, b.y)
    point2_ext(lhs, b)
}

/// The displacement from the starting point to a parameter-line point is the scaled endpoint displacement.
theorem point2_sub_param_line_start[T: CommRing](a: Point2[T], b: Point2[T], t: T) {
    a.param_line(b, t).sub(a) = b.sub(a).smul(t)
} by {
    let w = b.sub(a).smul(t)
    let lhs = a.param_line(b, t).sub(a)
    lhs.x = a.x + w.x - a.x
    a.x + w.x - a.x = w.x
    lhs.x = w.x
    lhs.y = a.y + w.y - a.y
    a.y + w.y - a.y = w.y
    lhs.y = w.y
    point2_ext(lhs, w)
}

/// Parameter-line points are collinear with their endpoints.
theorem point2_param_line_collinear[T: CommRing](a: Point2[T], b: Point2[T], t: T) {
    a.collinear(b, a.param_line(b, t))
} by {
    let u = b.sub(a)
    point2_sub_param_line_start(a, b, t)
    a.orientation(b, a.param_line(b, t)) = u.cross(u.smul(t))
    point2_cross_smul_right(t, u, u)
    point2_cross_self(u)
    t * u.cross(u) = T.0
}

/// Translating a parameter-line point is the parameter-line point of translated endpoints.
theorem point2_param_line_translate[T: CommRing](a: Point2[T], b: Point2[T], t: T, v: Point2[T]) {
    a.param_line(b, t).translate(v) = a.translate(v).param_line(b.translate(v), t)
} by {
    let u = b.sub(a)
    let w = u.smul(t)
    point2_sub_translate(b, a, v)
    point2_add_right_comm(a, w, v)
}

/// Reversing the endpoints of a parameter line replaces `t` by `1 - t`.
theorem point2_param_line_reverse[T: CommRing](a: Point2[T], b: Point2[T], t: T) {
    b.param_line(a, T.1 - t) = a.param_line(b, t)
} by {
    let lhs = b.param_line(a, T.1 - t)
    let rhs = a.param_line(b, t)
    lhs.x = b.x + (a.sub(b).smul(T.1 - t)).x
    (a.sub(b).smul(T.1 - t)).x = (T.1 - t) * a.sub(b).x
    a.sub(b).x = a.x - b.x
    lhs.x = b.x + (T.1 - t) * (a.x - b.x)
    rhs.x = a.x + (b.sub(a).smul(t)).x
    (b.sub(a).smul(t)).x = t * b.sub(a).x
    b.sub(a).x = b.x - a.x
    rhs.x = a.x + t * (b.x - a.x)
    point2_scalar_param_line_reverse(a.x, b.x, t)
    lhs.x = rhs.x
    lhs.y = b.y + (a.sub(b).smul(T.1 - t)).y
    (a.sub(b).smul(T.1 - t)).y = (T.1 - t) * a.sub(b).y
    a.sub(b).y = a.y - b.y
    lhs.y = b.y + (T.1 - t) * (a.y - b.y)
    rhs.y = a.y + (b.sub(a).smul(t)).y
    (b.sub(a).smul(t)).y = t * b.sub(a).y
    b.sub(a).y = b.y - a.y
    rhs.y = a.y + t * (b.y - a.y)
    point2_scalar_param_line_reverse(a.y, b.y, t)
    lhs.y = rhs.y
    point2_ext(lhs, rhs)
}
