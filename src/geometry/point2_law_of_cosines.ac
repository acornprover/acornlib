from comm_ring import CommRing
from geometry.point2 import Point2
from geometry.point2_algebra import point2_dot_comm, point2_dist_sq_comm,
    point2_norm_sq_sub_expansion
from geometry.point2_heron import point2_sub_same_base,
    point2_sub_eq_sub_imp_sub_eq, point2_sub_sub_eq_sub_add

/// Law of cosines in squared-distance / dot-product form at vertex `a`.
theorem point2_law_of_cosines_dist_sq[T: CommRing](a: Point2[T], b: Point2[T], c: Point2[T]) {
    b.dist_sq(c) =
        a.dist_sq(b) + a.dist_sq(c) -
        b.sub(a).dot(c.sub(a)) - b.sub(a).dot(c.sub(a))
} by {
    let u = b.sub(a)
    let v = c.sub(a)

    point2_sub_same_base(a, b, c)
    v.sub(u) = c.sub(b)

    point2_dist_sq_comm(b, c)
    b.dist_sq(c) = c.dist_sq(b)
    c.dist_sq(b) = c.sub(b).norm_sq
    c.sub(b).norm_sq = v.sub(u).norm_sq
    b.dist_sq(c) = v.sub(u).norm_sq

    point2_norm_sq_sub_expansion(v, u)
    v.sub(u).norm_sq = v.norm_sq + u.norm_sq - v.dot(u) - v.dot(u)

    point2_dot_comm(v, u)
    v.dot(u) = u.dot(v)
    v.norm_sq + u.norm_sq - v.dot(u) - v.dot(u) =
        u.norm_sq + v.norm_sq - u.dot(v) - u.dot(v)
    b.dist_sq(c) = u.norm_sq + v.norm_sq - u.dot(v) - u.dot(v)

    point2_dist_sq_comm(a, b)
    a.dist_sq(b) = b.dist_sq(a)
    b.dist_sq(a) = b.sub(a).norm_sq
    a.dist_sq(b) = u.norm_sq

    point2_dist_sq_comm(a, c)
    a.dist_sq(c) = c.dist_sq(a)
    c.dist_sq(a) = c.sub(a).norm_sq
    a.dist_sq(c) = v.norm_sq

    u.dot(v) = b.sub(a).dot(c.sub(a))
    b.dist_sq(c) =
        a.dist_sq(b) + a.dist_sq(c) -
        b.sub(a).dot(c.sub(a)) - b.sub(a).dot(c.sub(a))
}

/// Twice the dot product at vertex `a` is determined by the three squared side lengths.
theorem point2_double_dot_from_dist_sq[T: CommRing](a: Point2[T], b: Point2[T], c: Point2[T]) {
    b.sub(a).dot(c.sub(a)) + b.sub(a).dot(c.sub(a)) =
        a.dist_sq(b) + a.dist_sq(c) - b.dist_sq(c)
} by {
    let d = b.sub(a).dot(c.sub(a))
    let sum_sq = a.dist_sq(b) + a.dist_sq(c)

    point2_law_of_cosines_dist_sq(a, b, c)
    b.dist_sq(c) = sum_sq - d - d

    point2_sub_sub_eq_sub_add(sum_sq, d, d)
    sum_sq - d - d = sum_sq - (d + d)
    b.dist_sq(c) = sum_sq - (d + d)

    point2_sub_eq_sub_imp_sub_eq(sum_sq, d + d, b.dist_sq(c))
    sum_sq - b.dist_sq(c) = d + d
    d + d = sum_sq - b.dist_sq(c)
    b.sub(a).dot(c.sub(a)) + b.sub(a).dot(c.sub(a)) =
        a.dist_sq(b) + a.dist_sq(c) - b.dist_sq(c)
}
