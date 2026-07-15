from comm_ring import CommRing
from ordered_field import OrderedField
from geometry.point2 import Point2, point2_orientation_same_first,
    point2_orientation_same_second, point2_orientation_same_third
from geometry.point2_algebra import point2_sub_reverse_neg,
    point2_sub_through, point2_cross_self, point2_cross_swap,
    point2_cross_add_right, point2_cross_neg_right

/// Three points are collinear when the first two points coincide.
theorem point2_collinear_same_first[T: CommRing](a: Point2[T], b: Point2[T]) {
    a.collinear(a, b)
} by {
    point2_orientation_same_first(a, b)
}

/// Three points are collinear when the first and third points coincide.
theorem point2_collinear_same_second[T: CommRing](a: Point2[T], b: Point2[T]) {
    a.collinear(b, a)
} by {
    point2_orientation_same_second(a, b)
}

/// Three points are collinear when the last two points coincide.
theorem point2_collinear_same_third[T: CommRing](a: Point2[T], b: Point2[T]) {
    a.collinear(b, b)
} by {
    point2_orientation_same_third(a, b)
}

/// Swapping the last two points negates orientation.
theorem point2_orientation_swap_neg[T: CommRing](a: Point2[T], b: Point2[T], c: Point2[T]) {
    a.orientation(b, c) = -a.orientation(c, b)
} by {
    point2_cross_swap(b.sub(a), c.sub(a))
}

/// Orientation is invariant under cyclic permutation of the three points.
theorem point2_orientation_rotate[T: CommRing](a: Point2[T], b: Point2[T], c: Point2[T]) {
    a.orientation(b, c) = b.orientation(c, a)
} by {
    let u = b.sub(a)
    let v = c.sub(b)
    point2_sub_through(a, b, c)
    point2_cross_add_right(u, u, v)
    point2_cross_self(u)

    point2_sub_reverse_neg(a, b)
    point2_cross_neg_right(v, u)
    point2_cross_swap(u, v)
}

/// Swapping the first two points negates orientation.
theorem point2_orientation_swap_first_second_neg[T: CommRing](a: Point2[T], b: Point2[T], c: Point2[T]) {
    a.orientation(b, c) = -b.orientation(a, c)
} by {
    point2_orientation_rotate(b, a, c)
    point2_orientation_swap_neg(a, b, c)
}

/// Swapping the first and third points negates orientation.
theorem point2_orientation_swap_first_third_neg[T: CommRing](a: Point2[T], b: Point2[T], c: Point2[T]) {
    a.orientation(b, c) = -c.orientation(b, a)
} by {
    point2_orientation_rotate(c, b, a)
    point2_orientation_swap_first_second_neg(a, b, c)
}

/// Swapping the last two points negates orientation.
theorem point2_orientation_swap_second_third_neg[T: CommRing](a: Point2[T], b: Point2[T], c: Point2[T]) {
    a.orientation(b, c) = -a.orientation(c, b)
} by {
    point2_orientation_swap_neg(a, b, c)
}

/// Collinearity is invariant under cyclic permutation of the three points.
theorem point2_collinear_rotate[T: CommRing](a: Point2[T], b: Point2[T], c: Point2[T]) {
    a.collinear(b, c) = b.collinear(c, a)
} by {
    point2_orientation_rotate(a, b, c)
}

/// Collinearity is invariant under swapping the first two points.
theorem point2_collinear_swap_first_second[T: CommRing](a: Point2[T], b: Point2[T], c: Point2[T]) {
    a.collinear(b, c) = b.collinear(a, c)
} by {
    if a.collinear(b, c) {
        point2_orientation_swap_first_second_neg(a, b, c)
        -b.orientation(a, c) = T.0
        b.orientation(a, c) = T.0
        b.collinear(a, c)
    }
    if b.collinear(a, c) {
        point2_orientation_swap_first_second_neg(a, b, c)
        -b.orientation(a, c) = -T.0
        -T.0 = T.0
        a.collinear(b, c)
    }
}

/// Collinearity is invariant under swapping the first and third points.
theorem point2_collinear_swap_first_third[T: CommRing](a: Point2[T], b: Point2[T], c: Point2[T]) {
    a.collinear(b, c) = c.collinear(b, a)
} by {
    if a.collinear(b, c) {
        point2_orientation_swap_first_third_neg(a, b, c)
        -c.orientation(b, a) = T.0
        c.collinear(b, a)
    }
    if c.collinear(b, a) {
        point2_orientation_swap_first_third_neg(a, b, c)
        -c.orientation(b, a) = -T.0
        -T.0 = T.0
        a.collinear(b, c)
    }
}

/// Collinearity is invariant under swapping the last two points.
theorem point2_collinear_swap_second_third[T: CommRing](a: Point2[T], b: Point2[T], c: Point2[T]) {
    a.collinear(b, c) = a.collinear(c, b)
} by {
    if a.collinear(b, c) {
        point2_orientation_swap_second_third_neg(a, b, c)
        -a.orientation(c, b) = T.0
        a.collinear(c, b)
    }
    if a.collinear(c, b) {
        point2_orientation_swap_second_third_neg(a, b, c)
        -a.orientation(c, b) = -T.0
        -T.0 = T.0
        a.collinear(b, c)
    }
}

/// A left turn is not collinear.
theorem point2_left_turn_not_collinear[T: OrderedField](a: Point2[T], b: Point2[T], c: Point2[T]) {
    a.left_turn(b, c) implies not a.collinear(b, c)
} by {
    if a.left_turn(b, c) {
        a.orientation(b, c) != T.0
        not a.collinear(b, c)
    }
}

/// A right turn is not collinear.
theorem point2_right_turn_not_collinear[T: OrderedField](a: Point2[T], b: Point2[T], c: Point2[T]) {
    a.right_turn(b, c) implies not a.collinear(b, c)
} by {
    if a.right_turn(b, c) {
        a.orientation(b, c) != T.0
        not a.collinear(b, c)
    }
}
