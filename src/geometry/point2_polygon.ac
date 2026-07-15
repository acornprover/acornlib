from ordered_field import OrderedField
from list import List
from geometry.point2 import Point2
from geometry.point2_triangle import point2_triangle_area2_same_first,
    point2_triangle_area2_translate

/// The list obtained by translating every point by a fixed displacement.
define point2_translate_points[T: OrderedField](points: List[Point2[T]], v: Point2[T]) -> List[Point2[T]] {
    match points {
        List.nil {
            List.nil[Point2[T]]
        }
        List.cons(head, tail) {
            List.cons(head.translate(v), point2_translate_points(tail, v))
        }
    }
}

/// The signed doubled area swept by a point chain with a fixed anchor and previous vertex.
define point2_polygon_chain_area2[T: OrderedField](anchor: Point2[T], previous: Point2[T], rest: List[Point2[T]]) -> T {
    match rest {
        List.nil {
            T.0
        }
        List.cons(next, tail) {
            anchor.triangle_area2(previous, next) + point2_polygon_chain_area2(anchor, next, tail)
        }
    }
}

/// The signed doubled area of a polygonal point list, triangulated from its first point.
define point2_polygon_area2[T: OrderedField](points: List[Point2[T]]) -> T {
    match points {
        List.nil {
            T.0
        }
        List.cons(first, tail) {
            point2_polygon_chain_area2(first, first, tail)
        }
    }
}

/// True when a polygonal point list has positive signed doubled area.
define point2_polygon_ccw[T: OrderedField](points: List[Point2[T]]) -> Bool {
    point2_polygon_area2(points) > T.0
}

/// True when a polygonal point list has negative signed doubled area.
define point2_polygon_cw[T: OrderedField](points: List[Point2[T]]) -> Bool {
    point2_polygon_area2(points) < T.0
}

/// True when a polygonal point list has zero signed doubled area.
define point2_polygon_area2_zero[T: OrderedField](points: List[Point2[T]]) -> Bool {
    point2_polygon_area2(points) = T.0
}

/// Translating an empty point list gives the empty point list.
theorem point2_translate_points_nil[T: OrderedField](v: Point2[T]) {
    point2_translate_points(List.nil[Point2[T]], v) = List.nil[Point2[T]]
}

/// Translating a nonempty point list translates its head and tail separately.
theorem point2_translate_points_cons[T: OrderedField](head: Point2[T], tail: List[Point2[T]], v: Point2[T]) {
    point2_translate_points(List.cons(head, tail), v) =
    List.cons(head.translate(v), point2_translate_points(tail, v))
}

/// An empty chain has zero signed doubled area.
theorem point2_polygon_chain_area2_nil[T: OrderedField](anchor: Point2[T], previous: Point2[T]) {
    point2_polygon_chain_area2(anchor, previous, List.nil[Point2[T]]) = T.0
}

/// A one-edge chain has the signed doubled area of the corresponding triangle.
theorem point2_polygon_chain_area2_single[T: OrderedField](anchor: Point2[T], previous: Point2[T], next: Point2[T]) {
    point2_polygon_chain_area2(anchor, previous, List.cons(next, List.nil[Point2[T]])) =
    anchor.triangle_area2(previous, next)
} by {
    point2_polygon_chain_area2(anchor, next, List.nil[Point2[T]]) = T.0
}

/// The empty polygonal point list has zero signed doubled area.
theorem point2_polygon_area2_nil[T: OrderedField] {
    point2_polygon_area2(List.nil[Point2[T]]) = T.0
}

/// A singleton polygonal point list has zero signed doubled area.
theorem point2_polygon_area2_single[T: OrderedField](a: Point2[T]) {
    point2_polygon_area2(List.cons(a, List.nil[Point2[T]])) = T.0
}

/// A two-point polygonal point list has zero signed doubled area.
theorem point2_polygon_area2_pair[T: OrderedField](a: Point2[T], b: Point2[T]) {
    point2_polygon_area2(List.cons(a, List.cons(b, List.nil[Point2[T]]))) = T.0
} by {
    point2_triangle_area2_same_first(a, b)
}

/// A three-point polygonal point list has the signed doubled area of its triangle.
theorem point2_polygon_area2_triple[T: OrderedField](a: Point2[T], b: Point2[T], c: Point2[T]) {
    point2_polygon_area2(List.cons(a, List.cons(b, List.cons(c, List.nil[Point2[T]])))) =
    a.triangle_area2(b, c)
} by {
    point2_triangle_area2_same_first(a, b)
    point2_polygon_chain_area2_single(a, b, c)
    point2_polygon_chain_area2(a, a, List.cons(b, List.cons(c, List.nil[Point2[T]]))) =
    a.triangle_area2(a, b) + point2_polygon_chain_area2(a, b, List.cons(c, List.nil[Point2[T]]))
}

/// A four-point polygonal point list is the sum of two fan triangles.
theorem point2_polygon_area2_quad[T: OrderedField](a: Point2[T], b: Point2[T], c: Point2[T], d: Point2[T]) {
    point2_polygon_area2(List.cons(a, List.cons(b, List.cons(c, List.cons(d, List.nil[Point2[T]]))))) =
    a.triangle_area2(b, c) + a.triangle_area2(c, d)
} by {
    point2_triangle_area2_same_first(a, b)
    point2_polygon_chain_area2_single(a, c, d)
    point2_polygon_chain_area2(a, a, List.cons(b, List.cons(c, List.cons(d, List.nil[Point2[T]])))) =
    a.triangle_area2(a, b) + point2_polygon_chain_area2(a, b, List.cons(c, List.cons(d, List.nil[Point2[T]])))
}

/// Translating every point in a chain preserves its signed doubled area.
theorem point2_polygon_chain_area2_translate[T: OrderedField](
    anchor: Point2[T],
    previous: Point2[T],
    rest: List[Point2[T]],
    v: Point2[T]
) {
    point2_polygon_chain_area2(anchor.translate(v), previous.translate(v), point2_translate_points(rest, v)) =
    point2_polygon_chain_area2(anchor, previous, rest)
} by {
    define p(xs: List[Point2[T]], prev: Point2[T]) -> Bool {
        point2_polygon_chain_area2(anchor.translate(v), prev.translate(v), point2_translate_points(xs, v)) =
        point2_polygon_chain_area2(anchor, prev, xs)
    }
    define q(xs: List[Point2[T]]) -> Bool {
        forall(prev: Point2[T]) { p(xs, prev) }
    }

    forall(prev: Point2[T]) {
        point2_polygon_chain_area2(anchor.translate(v), prev.translate(v), List.nil[Point2[T]]) = T.0
        p(List.nil[Point2[T]], prev)
    }
    q(List.nil[Point2[T]])

    forall(head: Point2[T], tail: List[Point2[T]]) {
        if q(tail) {
            forall(prev: Point2[T]) {
                point2_triangle_area2_translate(anchor, prev, head, v)
                p(tail, head)
                point2_polygon_chain_area2(anchor.translate(v), head.translate(v), point2_translate_points(tail, v)) =
                point2_polygon_chain_area2(anchor, head, tail)
                anchor.translate(v).triangle_area2(prev.translate(v), head.translate(v)) +
                point2_polygon_chain_area2(anchor.translate(v), head.translate(v), point2_translate_points(tail, v)) =
                anchor.triangle_area2(prev, head) + point2_polygon_chain_area2(anchor, head, tail)
                point2_polygon_chain_area2(anchor.translate(v), prev.translate(v), point2_translate_points(List.cons(head, tail), v)) =
                point2_polygon_chain_area2(anchor, prev, List.cons(head, tail))
                p(List.cons(head, tail), prev)
            }
            q(List.cons(head, tail))
        }
    }

    List.induction(q)
    q(rest)
    p(rest, previous)
}

/// Translating every point in a polygonal point list preserves its signed doubled area.
theorem point2_polygon_area2_translate[T: OrderedField](points: List[Point2[T]], v: Point2[T]) {
    point2_polygon_area2(point2_translate_points(points, v)) = point2_polygon_area2(points)
} by {
    match points {
        List.nil {
            point2_translate_points(points, v) = List.nil[Point2[T]]
            point2_polygon_area2(point2_translate_points(points, v)) = point2_polygon_area2(points)
        }
        List.cons(first, tail) {
            point2_translate_points(points, v) = List.cons(first.translate(v), point2_translate_points(tail, v))
            point2_polygon_chain_area2_translate(first, first, tail, v)
            point2_polygon_area2(point2_translate_points(points, v)) = point2_polygon_area2(points)
        }
    }
}

/// Translating every point preserves positive signed area.
theorem point2_polygon_ccw_translate[T: OrderedField](points: List[Point2[T]], v: Point2[T]) {
    point2_polygon_ccw(point2_translate_points(points, v)) = point2_polygon_ccw(points)
} by {
    point2_polygon_area2_translate(points, v)
}

/// Translating every point preserves negative signed area.
theorem point2_polygon_cw_translate[T: OrderedField](points: List[Point2[T]], v: Point2[T]) {
    point2_polygon_cw(point2_translate_points(points, v)) = point2_polygon_cw(points)
} by {
    point2_polygon_area2_translate(points, v)
}

/// Translating every point preserves the zero signed-area predicate.
theorem point2_polygon_area2_zero_translate[T: OrderedField](points: List[Point2[T]], v: Point2[T]) {
    point2_polygon_area2_zero(point2_translate_points(points, v)) = point2_polygon_area2_zero(points)
} by {
    point2_polygon_area2_translate(points, v)
}
