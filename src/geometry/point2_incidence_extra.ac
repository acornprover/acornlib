/// Extra computational and transport endpoints for two-dimensional point incidence.

from comm_ring import CommRing
from ordered_field import OrderedField
from geometry.point2 import Point2, point2_zero
from geometry.point2_algebra import point2_dist_sq_comm, point2_dist_sq_zero_right
from geometry.point2_orientation import point2_collinear_rotate,
    point2_collinear_swap_first_second, point2_orientation_rotate
from geometry.point2_segment import point2_on_line_iff_collinear, point2_on_line_translate,
    point2_on_segment_swap, point2_on_segment_translate,
    point2_between_iff_on_segment, point2_between_translate
from geometry.point2_ray import point2_on_ray_imp_on_line, point2_on_ray_translate
from geometry.point2_circle import point2_on_circle_translate, point2_same_circle_translate,
    point2_on_circle_center_dist, point2_center_dist_imp_on_circle,
    point2_same_circle_dist_sq_eq, point2_same_circle_first_center_dist,
    point2_same_circle_second_center_dist
from geometry.point2_side import point2_left_of_line_translate, point2_right_of_line_translate,
    point2_same_strict_side_translate, point2_opposite_strict_sides_translate
from geometry.point2_halfplane import point2_open_left_halfplane_translate,
    point2_open_right_halfplane_translate, point2_closed_left_halfplane_translate,
    point2_closed_right_halfplane_translate, point2_halfplane_boundary_translate,
    point2_same_closed_halfplane_translate, point2_opposite_open_halfplanes_translate

/// Dot product in coordinate normal form.
theorem point2_dot_coordinate_formula[T: CommRing](p: Point2[T], q: Point2[T]) {
    p.dot(q) = p.x * q.x + p.y * q.y
}

/// Cross product in coordinate normal form.
theorem point2_cross_coordinate_formula[T: CommRing](p: Point2[T], q: Point2[T]) {
    p.cross(q) = p.x * q.y - p.y * q.x
}

/// Squared norm in coordinate normal form.
theorem point2_norm_sq_coordinate_formula[T: CommRing](p: Point2[T]) {
    p.norm_sq = p.x * p.x + p.y * p.y
} by {
}

/// Squared distance in coordinate normal form.
theorem point2_dist_sq_coordinate_formula[T: CommRing](p: Point2[T], q: Point2[T]) {
    p.dist_sq(q) = (p.x - q.x) * (p.x - q.x) + (p.y - q.y) * (p.y - q.y)
} by {
    let d = p.sub(q)
    d.norm_sq = d.x * d.x + d.y * d.y
}

/// Orientation determinant in coordinate normal form.
theorem point2_orientation_coordinate_formula[T: CommRing](a: Point2[T], b: Point2[T], c: Point2[T]) {
    a.orientation(b, c) = (b.x - a.x) * (c.y - a.y) - (b.y - a.y) * (c.x - a.x)
} by {
    let u = b.sub(a)
    let v = c.sub(a)
    u.cross(v) = u.x * v.y - u.y * v.x
}

/// Squared distance from the origin on the left is the squared norm.
theorem point2_dist_sq_zero_left[T: CommRing](p: Point2[T]) {
    point2_zero[T].dist_sq(p) = p.norm_sq
} by {
    point2_dist_sq_comm(point2_zero[T], p)
    point2_dist_sq_zero_right(p)
}

/// Left-turn membership is invariant under cyclic rotation.
theorem point2_left_turn_rotate[T: OrderedField](a: Point2[T], b: Point2[T], c: Point2[T]) {
    a.left_turn(b, c) = b.left_turn(c, a)
} by {
    point2_orientation_rotate(a, b, c)
}

/// Right-turn membership is invariant under cyclic rotation.
theorem point2_right_turn_rotate[T: OrderedField](a: Point2[T], b: Point2[T], c: Point2[T]) {
    a.right_turn(b, c) = b.right_turn(c, a)
} by {
    point2_orientation_rotate(a, b, c)
}

/// Line membership is invariant under cyclic rotation of the three collinear arguments.
theorem point2_on_line_rotate[T: OrderedField](a: Point2[T], b: Point2[T], p: Point2[T]) {
    a.on_line(b, p) = b.on_line(p, a)
} by {
    point2_on_line_iff_collinear(a, b, p)
    point2_on_line_iff_collinear(b, p, a)
    point2_collinear_rotate(a, b, p)
}

/// A line can be viewed with the queried point as the first defining point.
theorem point2_on_line_point_first[T: OrderedField](a: Point2[T], b: Point2[T], p: Point2[T]) {
    p.on_line(a, b) = a.on_line(b, p)
} by {
    point2_on_line_iff_collinear(p, a, b)
    point2_on_line_iff_collinear(a, b, p)
    point2_collinear_rotate(p, a, b)
}

/// Swapping the first two collinearity arguments is the line endpoint swap.
theorem point2_on_line_endpoint_swap_from_collinear[T: OrderedField](a: Point2[T], b: Point2[T], p: Point2[T]) {
    a.on_line(b, p) = b.on_line(a, p)
} by {
    point2_on_line_iff_collinear(a, b, p)
    point2_on_line_iff_collinear(b, a, p)
    point2_collinear_swap_first_second(a, b, p)
}

/// Translating a line incidence forward preserves membership.
theorem point2_on_line_translate_forward[T: OrderedField](a: Point2[T], b: Point2[T], p: Point2[T], v: Point2[T]) {
    a.on_line(b, p) implies a.translate(v).on_line(b.translate(v), p.translate(v))
} by {
    if a.on_line(b, p) {
        point2_on_line_translate(a, b, p, v)
    }
}

/// Translating a line incidence backward reflects membership.
theorem point2_on_line_translate_backward[T: OrderedField](a: Point2[T], b: Point2[T], p: Point2[T], v: Point2[T]) {
    a.translate(v).on_line(b.translate(v), p.translate(v)) implies a.on_line(b, p)
} by {
    if a.translate(v).on_line(b.translate(v), p.translate(v)) {
        point2_on_line_translate(a, b, p, v)
    }
}

/// Segment endpoint symmetry as an implication.
theorem point2_on_segment_swap_forward[T: OrderedField](a: Point2[T], b: Point2[T], p: Point2[T]) {
    a.on_segment(b, p) implies b.on_segment(a, p)
} by {
    if a.on_segment(b, p) {
        point2_on_segment_swap(a, b, p)
    }
}

/// Translating a segment incidence backward reflects membership.
theorem point2_on_segment_translate_backward[T: OrderedField](a: Point2[T], b: Point2[T], p: Point2[T], v: Point2[T]) {
    a.translate(v).on_segment(b.translate(v), p.translate(v)) implies a.on_segment(b, p)
} by {
    if a.translate(v).on_segment(b.translate(v), p.translate(v)) {
        point2_on_segment_translate(a, b, p, v)
    }
}

/// Betweenness is segment membership as an implication.
theorem point2_between_imp_on_segment_extra[T: OrderedField](a: Point2[T], p: Point2[T], b: Point2[T]) {
    a.between(p, b) implies a.on_segment(b, p)
} by {
    if a.between(p, b) {
        point2_between_iff_on_segment(a, p, b)
        a.on_segment(b, p)
    }
}

/// Segment membership is betweenness as an implication.
theorem point2_on_segment_imp_between_extra[T: OrderedField](a: Point2[T], b: Point2[T], p: Point2[T]) {
    a.on_segment(b, p) implies a.between(p, b)
} by {
    if a.on_segment(b, p) {
        point2_between_iff_on_segment(a, p, b)
        a.between(p, b)
    }
}

/// Translating betweenness forward preserves membership.
theorem point2_between_translate_forward_extra[T: OrderedField](a: Point2[T], p: Point2[T], b: Point2[T], v: Point2[T]) {
    a.between(p, b) implies a.translate(v).between(p.translate(v), b.translate(v))
} by {
    if a.between(p, b) {
        point2_between_translate(a, p, b, v)
    }
}

/// Translating betweenness backward reflects membership.
theorem point2_between_translate_backward_extra[T: OrderedField](a: Point2[T], p: Point2[T], b: Point2[T], v: Point2[T]) {
    a.translate(v).between(p.translate(v), b.translate(v)) implies a.between(p, b)
} by {
    if a.translate(v).between(p.translate(v), b.translate(v)) {
        point2_between_translate(a, p, b, v)
    }
}

/// Ray membership implies collinearity with the defining line.
theorem point2_on_ray_imp_collinear[T: OrderedField](a: Point2[T], b: Point2[T], p: Point2[T]) {
    a.on_ray(b, p) implies a.collinear(b, p)
} by {
    if a.on_ray(b, p) {
        point2_on_ray_imp_on_line(a, b, p)
        point2_on_line_iff_collinear(a, b, p)
        a.collinear(b, p)
    }
}

/// Translating a ray incidence forward preserves membership.
theorem point2_on_ray_translate_forward_extra[T: OrderedField](a: Point2[T], b: Point2[T], p: Point2[T], v: Point2[T]) {
    a.on_ray(b, p) implies a.translate(v).on_ray(b.translate(v), p.translate(v))
} by {
    if a.on_ray(b, p) {
        point2_on_ray_translate(a, b, p, v)
    }
}

/// Translating a ray incidence backward reflects membership.
theorem point2_on_ray_translate_backward_extra[T: OrderedField](a: Point2[T], b: Point2[T], p: Point2[T], v: Point2[T]) {
    a.translate(v).on_ray(b.translate(v), p.translate(v)) implies a.on_ray(b, p)
} by {
    if a.translate(v).on_ray(b.translate(v), p.translate(v)) {
        point2_on_ray_translate(a, b, p, v)
    }
}

/// Circle membership may be characterized using center-to-point squared distance.
theorem point2_on_circle_center_dist_iff[T: OrderedField](center: Point2[T], radius_sq: T, p: Point2[T]) {
    center.on_circle(radius_sq, p) = (center.dist_sq(p) = radius_sq)
} by {
    if center.on_circle(radius_sq, p) {
        point2_on_circle_center_dist(center, radius_sq, p)
        center.dist_sq(p) = radius_sq
    }
    if center.dist_sq(p) = radius_sq {
        point2_center_dist_imp_on_circle(center, radius_sq, p)
        center.on_circle(radius_sq, p)
    }
}

/// Translating a circle membership backward reflects membership.
theorem point2_on_circle_translate_backward[T: OrderedField](center: Point2[T], radius_sq: T, p: Point2[T], v: Point2[T]) {
    center.translate(v).on_circle(radius_sq, p.translate(v)) implies center.on_circle(radius_sq, p)
} by {
    if center.translate(v).on_circle(radius_sq, p.translate(v)) {
        point2_on_circle_translate(center, radius_sq, p, v)
    }
}

/// Translating same-circle membership backward reflects membership.
theorem point2_same_circle_translate_backward[T: OrderedField](center: Point2[T], radius_sq: T, p: Point2[T], q: Point2[T], v: Point2[T]) {
    center.translate(v).same_circle(radius_sq, p.translate(v), q.translate(v)) implies
    center.same_circle(radius_sq, p, q)
} by {
    if center.translate(v).same_circle(radius_sq, p.translate(v), q.translate(v)) {
        point2_same_circle_translate(center, radius_sq, p, q, v)
    }
}

/// Same-circle membership gives equal point-to-center squared distances.
theorem point2_same_circle_point_center_dist_eq[T: OrderedField](center: Point2[T], radius_sq: T, p: Point2[T], q: Point2[T]) {
    center.same_circle(radius_sq, p, q) implies p.dist_sq(center) = q.dist_sq(center)
} by {
    if center.same_circle(radius_sq, p, q) {
        point2_same_circle_dist_sq_eq(center, radius_sq, p, q)
    }
}

/// Same-circle membership gives equal center-to-point squared distances.
theorem point2_same_circle_center_point_dist_eq[T: OrderedField](center: Point2[T], radius_sq: T, p: Point2[T], q: Point2[T]) {
    center.same_circle(radius_sq, p, q) implies center.dist_sq(p) = center.dist_sq(q)
} by {
    if center.same_circle(radius_sq, p, q) {
        point2_same_circle_first_center_dist(center, radius_sq, p, q)
        point2_same_circle_second_center_dist(center, radius_sq, p, q)
        center.dist_sq(q) = radius_sq
    }
}

/// Translating strict left-of-line membership forward preserves membership.
theorem point2_left_of_line_translate_forward_extra[T: OrderedField](a: Point2[T], b: Point2[T], p: Point2[T], v: Point2[T]) {
    a.left_of_line(b, p) implies a.translate(v).left_of_line(b.translate(v), p.translate(v))
} by {
    if a.left_of_line(b, p) {
        point2_left_of_line_translate(a, b, p, v)
    }
}

/// Translating strict right-of-line membership forward preserves membership.
theorem point2_right_of_line_translate_forward_extra[T: OrderedField](a: Point2[T], b: Point2[T], p: Point2[T], v: Point2[T]) {
    a.right_of_line(b, p) implies a.translate(v).right_of_line(b.translate(v), p.translate(v))
} by {
    if a.right_of_line(b, p) {
        point2_right_of_line_translate(a, b, p, v)
    }
}

/// Translating strict same-side membership backward reflects membership.
theorem point2_same_strict_side_translate_backward[T: OrderedField](a: Point2[T], b: Point2[T], p: Point2[T], q: Point2[T], v: Point2[T]) {
    a.translate(v).same_strict_side(b.translate(v), p.translate(v), q.translate(v)) implies
    a.same_strict_side(b, p, q)
} by {
    if a.translate(v).same_strict_side(b.translate(v), p.translate(v), q.translate(v)) {
        point2_same_strict_side_translate(a, b, p, q, v)
    }
}

/// Translating strict opposite-side membership backward reflects membership.
theorem point2_opposite_strict_sides_translate_backward[T: OrderedField](a: Point2[T], b: Point2[T], p: Point2[T], q: Point2[T], v: Point2[T]) {
    a.translate(v).opposite_strict_sides(b.translate(v), p.translate(v), q.translate(v)) implies
    a.opposite_strict_sides(b, p, q)
} by {
    if a.translate(v).opposite_strict_sides(b.translate(v), p.translate(v), q.translate(v)) {
        point2_opposite_strict_sides_translate(a, b, p, q, v)
    }
}

/// Translating open-left half-plane membership backward reflects membership.
theorem point2_open_left_halfplane_translate_backward_extra[T: OrderedField](a: Point2[T], b: Point2[T], p: Point2[T], v: Point2[T]) {
    a.translate(v).in_open_left_halfplane(b.translate(v), p.translate(v)) implies
    a.in_open_left_halfplane(b, p)
} by {
    if a.translate(v).in_open_left_halfplane(b.translate(v), p.translate(v)) {
        point2_open_left_halfplane_translate(a, b, p, v)
    }
}

/// Translating open-right half-plane membership backward reflects membership.
theorem point2_open_right_halfplane_translate_backward_extra[T: OrderedField](a: Point2[T], b: Point2[T], p: Point2[T], v: Point2[T]) {
    a.translate(v).in_open_right_halfplane(b.translate(v), p.translate(v)) implies
    a.in_open_right_halfplane(b, p)
} by {
    if a.translate(v).in_open_right_halfplane(b.translate(v), p.translate(v)) {
        point2_open_right_halfplane_translate(a, b, p, v)
    }
}

/// Translating closed-left half-plane membership backward reflects membership.
theorem point2_closed_left_halfplane_translate_backward_extra[T: OrderedField](a: Point2[T], b: Point2[T], p: Point2[T], v: Point2[T]) {
    a.translate(v).in_closed_left_halfplane(b.translate(v), p.translate(v)) implies
    a.in_closed_left_halfplane(b, p)
} by {
    if a.translate(v).in_closed_left_halfplane(b.translate(v), p.translate(v)) {
        point2_closed_left_halfplane_translate(a, b, p, v)
    }
}

/// Translating closed-right half-plane membership backward reflects membership.
theorem point2_closed_right_halfplane_translate_backward_extra[T: OrderedField](a: Point2[T], b: Point2[T], p: Point2[T], v: Point2[T]) {
    a.translate(v).in_closed_right_halfplane(b.translate(v), p.translate(v)) implies
    a.in_closed_right_halfplane(b, p)
} by {
    if a.translate(v).in_closed_right_halfplane(b.translate(v), p.translate(v)) {
        point2_closed_right_halfplane_translate(a, b, p, v)
    }
}

/// Translating half-plane boundary membership backward reflects membership.
theorem point2_halfplane_boundary_translate_backward_extra[T: OrderedField](a: Point2[T], b: Point2[T], p: Point2[T], v: Point2[T]) {
    a.translate(v).on_halfplane_boundary(b.translate(v), p.translate(v)) implies
    a.on_halfplane_boundary(b, p)
} by {
    if a.translate(v).on_halfplane_boundary(b.translate(v), p.translate(v)) {
        point2_halfplane_boundary_translate(a, b, p, v)
    }
}

/// Translating same closed-half-plane membership backward reflects membership.
theorem point2_same_closed_halfplane_translate_backward_extra[T: OrderedField](a: Point2[T], b: Point2[T], p: Point2[T], q: Point2[T], v: Point2[T]) {
    a.translate(v).same_closed_halfplane(b.translate(v), p.translate(v), q.translate(v)) implies
    a.same_closed_halfplane(b, p, q)
} by {
    if a.translate(v).same_closed_halfplane(b.translate(v), p.translate(v), q.translate(v)) {
        point2_same_closed_halfplane_translate(a, b, p, q, v)
    }
}

/// Translating opposite open-half-plane membership backward reflects membership.
theorem point2_opposite_open_halfplanes_translate_backward_extra[T: OrderedField](a: Point2[T], b: Point2[T], p: Point2[T], q: Point2[T], v: Point2[T]) {
    a.translate(v).opposite_open_halfplanes(b.translate(v), p.translate(v), q.translate(v)) implies
    a.opposite_open_halfplanes(b, p, q)
} by {
    if a.translate(v).opposite_open_halfplanes(b.translate(v), p.translate(v), q.translate(v)) {
        point2_opposite_open_halfplanes_translate(a, b, p, q, v)
    }
}
