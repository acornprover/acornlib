from nat import Nat
from ordered_field import OrderedField
from finite_set import FiniteSet, fs_image, finite_set_image_contains_eq,
    finite_set_subset_contains
from data.finite.finite_set_card import fs_card
from data.finite.finite_set_card_compare import fs_card_mono
from data.finite.finite_set_image_card import fs_card_image_le
from data.finite.finite_set_subset_intro import fs_subset_eq_intro
from geometry.point2 import Point2
from geometry.point2_concyclic import concyclic4, concyclic4_swap_first_two,
    concyclic4_rotate, concyclic4_swap_last_two

numerals Nat

/// The squared distance from a fixed point, as a function of the other point.
///
/// Named rather than written inline so it can appear in the image below, which is how the
/// set of realized distances is formed.
define dist_sq_from[T: OrderedField](p: Point2[T], q: Point2[T]) -> T {
    q.dist_sq(p)
}

/// The set of squared distances realized from a point to a finite planar set.
///
/// Squared distances rather than distances, since the ambient field need not have square
/// roots. Two points are at the same distance from `p` exactly when their squared distances
/// agree, so nothing about the counting changes.
define distances_from[T: OrderedField](
    p: Point2[T], s: FiniteSet[Point2[T]]
) -> FiniteSet[T] {
    fs_image(s, dist_sq_from(p))
}

/// Membership in the distance set.
theorem distances_from_contains_eq[T: OrderedField](
    p: Point2[T], s: FiniteSet[Point2[T]], t: T
) {
    distances_from(p, s).contains(t) = exists(q: Point2[T]) {
        s.contains(q) and t = dist_sq_from(p, q)
    }
} by {
    finite_set_image_contains_eq(s, dist_sq_from(p), t)
}

/// A member of the set realizes its distance.
theorem distances_from_contains[T: OrderedField](
    p: Point2[T], s: FiniteSet[Point2[T]], q: Point2[T]
) {
    s.contains(q) implies distances_from(p, s).contains(q.dist_sq(p))
} by {
    if s.contains(q) {
        dist_sq_from(p, q) = q.dist_sq(p)
        exists(r: Point2[T]) {
            s.contains(r) and q.dist_sq(p) = dist_sq_from(p, r)
        }
        distances_from_contains_eq(p, s, q.dist_sq(p))
        distances_from(p, s).contains(q.dist_sq(p))
    }
}

/// A realized distance comes from a member of the set.
theorem distances_from_witness[T: OrderedField](
    p: Point2[T], s: FiniteSet[Point2[T]], t: T
) {
    distances_from(p, s).contains(t) implies exists(q: Point2[T]) {
        s.contains(q) and t = q.dist_sq(p)
    }
} by {
    if distances_from(p, s).contains(t) {
        distances_from_contains_eq(p, s, t)
        exists(q: Point2[T]) {
            s.contains(q) and t = dist_sq_from(p, q)
        }
        let (q: Point2[T]) satisfy {
            s.contains(q) and t = dist_sq_from(p, q)
        }
        dist_sq_from(p, q) = q.dist_sq(p)
        t = q.dist_sq(p)
        exists(r: Point2[T]) {
            s.contains(r) and t = r.dist_sq(p)
        }
    }
}

/// The distance set grows with the point set.
theorem distances_from_mono[T: OrderedField](
    p: Point2[T], s: FiniteSet[Point2[T]], u: FiniteSet[Point2[T]]
) {
    s.subset_eq(u) implies distances_from(p, s).subset_eq(distances_from(p, u))
} by {
    if s.subset_eq(u) {
        forall(t: T) {
            if distances_from(p, s).contains(t) {
                distances_from_witness(p, s, t)
                let (q: Point2[T]) satisfy {
                    s.contains(q) and t = q.dist_sq(p)
                }
                finite_set_subset_contains(s, u, q)
                u.contains(q)
                distances_from_contains(p, u, q)
                distances_from(p, u).contains(q.dist_sq(p))
                distances_from(p, u).contains(t)
            }
            (distances_from(p, s).contains(t) implies distances_from(p, u).contains(t))
        }
        fs_subset_eq_intro(distances_from(p, s), distances_from(p, u))
        distances_from(p, s).subset_eq(distances_from(p, u))
    }
}

/// The number of distinct squared distances realized from a point.
define distance_count[T: OrderedField](
    p: Point2[T], s: FiniteSet[Point2[T]]
) -> Nat {
    fs_card(distances_from(p, s))
}

/// The distance count grows with the point set.
///
/// Enlarging the set of points can only add realized distances, never remove one.
theorem distance_count_mono[T: OrderedField](
    p: Point2[T], s: FiniteSet[Point2[T]], u: FiniteSet[Point2[T]]
) {
    s.subset_eq(u) implies distance_count(p, s) <= distance_count(p, u)
} by {
    if s.subset_eq(u) {
        distances_from_mono(p, s, u)
        distances_from(p, s).subset_eq(distances_from(p, u))
        fs_card_mono(distances_from(p, s), distances_from(p, u))
        fs_card(distances_from(p, s)) <= fs_card(distances_from(p, u))
        distance_count(p, s) <= distance_count(p, u)
    }
}

/// True when four points are distinct members of the set.
///
/// Named so that the no-four-concyclic condition below has a two-part body rather than a
/// ten-part one, which is what keeps its restriction to subsets tractable.
define four_distinct_in[T: OrderedField](
    s: FiniteSet[Point2[T]], a: Point2[T], b: Point2[T], c: Point2[T], d: Point2[T]
) -> Bool {
    s.contains(a) and s.contains(b) and s.contains(c) and s.contains(d)
        and a != b and a != c and a != d and b != c and b != d and c != d
}

/// The four points lie in the set and are pairwise different.
theorem four_distinct_in_apply[T: OrderedField](
    s: FiniteSet[Point2[T]], a: Point2[T], b: Point2[T], c: Point2[T], d: Point2[T]
) {
    four_distinct_in(s, a, b, c, d)
        implies s.contains(a) and s.contains(b) and s.contains(c) and s.contains(d)
            and a != b and a != c and a != d and b != c and b != d and c != d
} by {
    if four_distinct_in(s, a, b, c, d) {
        four_distinct_in(s, a, b, c, d) =
            (s.contains(a) and s.contains(b) and s.contains(c) and s.contains(d)
                and a != b and a != c and a != d and b != c and b != d and c != d)
        (s.contains(a) and s.contains(b) and s.contains(c) and s.contains(d)
            and a != b and a != c and a != d and b != c and b != d and c != d)
    }
}

/// Four distinct members give the condition.
theorem four_distinct_in_intro[T: OrderedField](
    s: FiniteSet[Point2[T]], a: Point2[T], b: Point2[T], c: Point2[T], d: Point2[T]
) {
    s.contains(a) and s.contains(b) and s.contains(c) and s.contains(d)
        and a != b and a != c and a != d and b != c and b != d and c != d
        implies four_distinct_in(s, a, b, c, d)
} by {
    if s.contains(a) and s.contains(b) and s.contains(c) and s.contains(d)
        and a != b and a != c and a != d and b != c and b != d and c != d {
        four_distinct_in(s, a, b, c, d) =
            (s.contains(a) and s.contains(b) and s.contains(c) and s.contains(d)
                and a != b and a != c and a != d and b != c and b != d and c != d)
        four_distinct_in(s, a, b, c, d)
    }
}

/// Four distinct members of a subset are four distinct members of the whole.
theorem four_distinct_in_of_subset[T: OrderedField](
    s: FiniteSet[Point2[T]], u: FiniteSet[Point2[T]],
    a: Point2[T], b: Point2[T], c: Point2[T], d: Point2[T]
) {
    s.subset_eq(u) and four_distinct_in(s, a, b, c, d)
        implies four_distinct_in(u, a, b, c, d)
} by {
    if s.subset_eq(u) and four_distinct_in(s, a, b, c, d) {
        four_distinct_in_apply(s, a, b, c, d)
        s.contains(a)
        finite_set_subset_contains(s, u, a)
        u.contains(a)
        s.contains(b)
        finite_set_subset_contains(s, u, b)
        u.contains(b)
        s.contains(c)
        finite_set_subset_contains(s, u, c)
        u.contains(c)
        s.contains(d)
        finite_set_subset_contains(s, u, d)
        u.contains(d)
        a != b
        a != c
        a != d
        b != c
        b != d
        c != d
        four_distinct_in_intro(u, a, b, c, d)
        four_distinct_in(u, a, b, c, d)
    }
}

/// True when no four distinct points of the set lie on a common circle.
///
/// The general-position condition that makes distance counting well behaved: with four points
/// concyclic, two pairs realize the same distance from the center for a reason that has
/// nothing to do with the rest of the configuration.
define no_four_concyclic[T: OrderedField](s: FiniteSet[Point2[T]]) -> Bool {
    forall(a: Point2[T], b: Point2[T], c: Point2[T], d: Point2[T]) {
        (four_distinct_in(s, a, b, c, d) implies not concyclic4(a, b, c, d))
    }
}

/// Four distinct points of such a set are not concyclic.
theorem no_four_concyclic_apply[T: OrderedField](
    s: FiniteSet[Point2[T]], a: Point2[T], b: Point2[T], c: Point2[T], d: Point2[T]
) {
    no_four_concyclic(s) and four_distinct_in(s, a, b, c, d)
        implies not concyclic4(a, b, c, d)
} by {
    if no_four_concyclic(s) and four_distinct_in(s, a, b, c, d) {
        no_four_concyclic(s) = forall(w: Point2[T], x: Point2[T], y: Point2[T], z: Point2[T]) {
            (four_distinct_in(s, w, x, y, z) implies not concyclic4(w, x, y, z))
        }
        forall(w: Point2[T], x: Point2[T], y: Point2[T], z: Point2[T]) {
            (four_distinct_in(s, w, x, y, z) implies not concyclic4(w, x, y, z))
        }
        (four_distinct_in(s, a, b, c, d) implies not concyclic4(a, b, c, d))
        not concyclic4(a, b, c, d)
    }
}

/// The pointwise condition gives the general-position condition.
theorem no_four_concyclic_intro[T: OrderedField](s: FiniteSet[Point2[T]]) {
    (forall(a: Point2[T], b: Point2[T], c: Point2[T], d: Point2[T]) {
        (four_distinct_in(s, a, b, c, d) implies not concyclic4(a, b, c, d))
    }) implies no_four_concyclic(s)
} by {
    if forall(a: Point2[T], b: Point2[T], c: Point2[T], d: Point2[T]) {
        (four_distinct_in(s, a, b, c, d) implies not concyclic4(a, b, c, d))
    } {
        no_four_concyclic(s) = forall(w: Point2[T], x: Point2[T], y: Point2[T], z: Point2[T]) {
            (four_distinct_in(s, w, x, y, z) implies not concyclic4(w, x, y, z))
        }
        no_four_concyclic(s)
    }
}

/// The general-position condition passes to subsets.
///
/// Four distinct points of a subset are four distinct points of the whole, so the whole
/// already forbids them.
theorem no_four_concyclic_of_subset[T: OrderedField](
    s: FiniteSet[Point2[T]], u: FiniteSet[Point2[T]]
) {
    s.subset_eq(u) and no_four_concyclic(u) implies no_four_concyclic(s)
} by {
    if s.subset_eq(u) and no_four_concyclic(u) {
        forall(a: Point2[T], b: Point2[T], c: Point2[T], d: Point2[T]) {
            if four_distinct_in(s, a, b, c, d) {
                four_distinct_in_of_subset(s, u, a, b, c, d)
                four_distinct_in(u, a, b, c, d)
                no_four_concyclic_apply(u, a, b, c, d)
                not concyclic4(a, b, c, d)
            }
            (four_distinct_in(s, a, b, c, d) implies not concyclic4(a, b, c, d))
        }
        no_four_concyclic_intro(s)
        no_four_concyclic(s)
    }
}

/// A point set realizes no more distances than it has points.
///
/// The distance set is the image of the point set, and a map can identify points but never
/// create new ones. This is the trivial upper bound that any distinct-distances question is
/// measured against.
theorem distance_count_le_card[T: OrderedField](
    p: Point2[T], s: FiniteSet[Point2[T]]
) {
    distance_count(p, s) <= fs_card(s)
} by {
    fs_card_image_le(s, dist_sq_from(p))
    fs_card(fs_image(s, dist_sq_from(p))) <= fs_card(s)
    distances_from(p, s) = fs_image(s, dist_sq_from(p))
    fs_card(distances_from(p, s)) <= fs_card(s)
    distance_count(p, s) <= fs_card(s)
}
