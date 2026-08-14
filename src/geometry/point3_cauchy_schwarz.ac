from algebra.add_ordered_group import add_le_add_left
from ordered_field import OrderedField, multiply_inequality_with_nonnegative_element
from algebra.field.field import mul_inverse_right
from geometry.point3 import Point3
from geometry.point3_algebra import point3_cross_norm_sq_identity,
    point3_norm_sq_nonneg

/// A nonnegative additive gap gives the corresponding order relation.
theorem point3_nonneg_gap_imp_le[T: OrderedField](a: T, b: T) {
    T.0 <= a - b implies b <= a
} by {
    if T.0 <= a - b {
        b + T.0 <= b + (a - b)
        b + (a - b) = a
        b <= a
    }
}

/// The squared Cauchy-Schwarz inequality in three dimensions.
theorem point3_cauchy_schwarz[T: OrderedField](u: Point3[T], v: Point3[T]) {
    u.dot(v) * u.dot(v) <= u.norm_sq * v.norm_sq
} by {
    point3_cross_norm_sq_identity(u, v)
    u.cross(v).norm_sq = u.norm_sq * v.norm_sq - u.dot(v) * u.dot(v)
    point3_norm_sq_nonneg(u.cross(v))
    u.cross(v).norm_sq >= T.0
    T.0 <= u.norm_sq * v.norm_sq - u.dot(v) * u.dot(v)
    point3_nonneg_gap_imp_le(u.norm_sq * v.norm_sq, u.dot(v) * u.dot(v))
    u.dot(v) * u.dot(v) <= u.norm_sq * v.norm_sq
}

/// The squared cosine of the angle between two coordinate vectors.
define point3_cos_angle_sq[T: OrderedField](u: Point3[T], v: Point3[T]) -> T {
    u.dot(v) * u.dot(v) * (u.norm_sq * v.norm_sq).inverse
}

/// The squared cosine of an angle between nonzero vectors is at most one.
theorem point3_cos_angle_sq_le_one[T: OrderedField](u: Point3[T], v: Point3[T]) {
    u.norm_sq * v.norm_sq != T.0 implies point3_cos_angle_sq(u, v) <= T.1
} by {
    if u.norm_sq * v.norm_sq != T.0 {
        point3_norm_sq_nonneg(u)
        point3_norm_sq_nonneg(v)
        u.norm_sq >= T.0
        v.norm_sq >= T.0
        T.0 <= u.norm_sq * v.norm_sq
        point3_cauchy_schwarz(u, v)
        u.dot(v) * u.dot(v) <= u.norm_sq * v.norm_sq
        T.0 <= (u.norm_sq * v.norm_sq).inverse
        multiply_inequality_with_nonnegative_element(
            u.dot(v) * u.dot(v),
            u.norm_sq * v.norm_sq,
            (u.norm_sq * v.norm_sq).inverse)
        (u.dot(v) * u.dot(v)) * (u.norm_sq * v.norm_sq).inverse <= (u.norm_sq * v.norm_sq) * (u.norm_sq * v.norm_sq).inverse
        mul_inverse_right[T](u.norm_sq * v.norm_sq)
        (u.norm_sq * v.norm_sq) * (u.norm_sq * v.norm_sq).inverse = T.1
        (u.dot(v) * u.dot(v)) * (u.norm_sq * v.norm_sq).inverse <= T.1
        point3_cos_angle_sq(u, v) =
            (u.dot(v) * u.dot(v)) * (u.norm_sq * v.norm_sq).inverse
        point3_cos_angle_sq(u, v) <= T.1
    }
}
