from ordered_field import OrderedField
from data.basic.logic import or_implies
from geometry.point2 import Point2
from geometry.point2_affine import point2_translate_translate,
    point2_translate_neg_right, point2_translate_cancel
from geometry.point2_segment import point2_on_segment_start,
    point2_on_segment_end, point2_on_segment_swap,
    point2_on_segment_translate, point2_on_segment_translate_forward
from geometry.point2_side import point2_opposite_strict_sides_translate,
    point2_opposite_strict_sides_reverse_line,
    point2_opposite_strict_sides_swap_points

/// A closed segment in the two-dimensional coordinate plane.
structure Point2Segment[T] {
    /// The first endpoint.
    start: Point2[T]

    /// The second endpoint.
    end: Point2[T]
}

/// A closed segment is determined by its endpoints.
theorem point2_segment_ext[T](s: Point2Segment[T], t: Point2Segment[T]) {
    s.start = t.start and s.end = t.end implies s = t
}

attributes Point2Segment[T: OrderedField] {
    /// True when a point lies on the closed segment.
    define contains(self, p: Point2[T]) -> Bool {
        self.start.on_segment(self.end, p)
    }

    /// The segment with its endpoints reversed.
    define reverse(self) -> Point2Segment[T] {
        Point2Segment.new(self.end, self.start)
    }

    /// The segment obtained by translating both endpoints.
    define translate(self, v: Point2[T]) -> Point2Segment[T] {
        Point2Segment.new(self.start.translate(v), self.end.translate(v))
    }

    /// True when two closed segments have a common point.
    define intersects(self, other: Point2Segment[T]) -> Bool {
        exists(p: Point2[T]) {
            self.contains(p) and other.contains(p)
        }
    }

    /// True when the two segments cross with endpoints on opposite strict sides.
    define properly_intersects(self, other: Point2Segment[T]) -> Bool {
        self.start.opposite_strict_sides(self.end, other.start, other.end) and
        other.start.opposite_strict_sides(other.end, self.start, self.end)
    }

    /// True when two segments have an endpoint in common.
    define shares_endpoint(self, other: Point2Segment[T]) -> Bool {
        self.start = other.start or self.start = other.end or
        self.end = other.start or self.end = other.end
    }
}

/// The start endpoint of a reversed segment is the original end endpoint.
theorem point2_segment_reverse_start[T: OrderedField](s: Point2Segment[T]) {
    s.reverse.start = s.end
}

/// The end endpoint of a reversed segment is the original start endpoint.
theorem point2_segment_reverse_end[T: OrderedField](s: Point2Segment[T]) {
    s.reverse.end = s.start
}

/// Reversing a segment twice gives the original segment.
theorem point2_segment_reverse_reverse[T: OrderedField](s: Point2Segment[T]) {
    s.reverse.reverse = s
} by {
    let lhs = s.reverse.reverse
    point2_segment_ext(lhs, s)
}

/// The start endpoint of a translated segment is the translated start endpoint.
theorem point2_segment_translate_start[T: OrderedField](s: Point2Segment[T], v: Point2[T]) {
    s.translate(v).start = s.start.translate(v)
}

/// The end endpoint of a translated segment is the translated end endpoint.
theorem point2_segment_translate_end[T: OrderedField](s: Point2Segment[T], v: Point2[T]) {
    s.translate(v).end = s.end.translate(v)
}

/// Successive translations of a segment compose by point addition.
theorem point2_segment_translate_translate[T: OrderedField](s: Point2Segment[T], u: Point2[T], v: Point2[T]) {
    s.translate(u).translate(v) = s.translate(u.add(v))
} by {
    let lhs = s.translate(u).translate(v)
    let rhs = s.translate(u.add(v))
    lhs.start = s.start.translate(u).translate(v)
    rhs.start = s.start.translate(u.add(v))
    point2_translate_translate(s.start, u, v)
    lhs.start = rhs.start
    lhs.end = s.end.translate(u).translate(v)
    rhs.end = s.end.translate(u.add(v))
    point2_translate_translate(s.end, u, v)
    lhs.end = rhs.end
    point2_segment_ext(lhs, rhs)
}

/// Translating a segment by a point and then by its negative gives the original segment.
theorem point2_segment_translate_neg_right[T: OrderedField](s: Point2Segment[T], v: Point2[T]) {
    s.translate(v).translate(v.neg) = s
} by {
    let lhs = s.translate(v).translate(v.neg)
    point2_translate_neg_right(s.start, v)
    lhs.start = s.start
    point2_translate_neg_right(s.end, v)
    point2_segment_ext(lhs, s)
}

/// Segment containment is closed-segment membership of the endpoints.
theorem point2_segment_contains_eq_on_segment[T: OrderedField](s: Point2Segment[T], p: Point2[T]) {
    s.contains(p) = s.start.on_segment(s.end, p)
}

/// The start endpoint lies on a segment.
theorem point2_segment_contains_start[T: OrderedField](s: Point2Segment[T]) {
    s.contains(s.start)
} by {
    point2_on_segment_start(s.start, s.end)
}

/// The end endpoint lies on a segment.
theorem point2_segment_contains_end[T: OrderedField](s: Point2Segment[T]) {
    s.contains(s.end)
} by {
    point2_on_segment_end(s.start, s.end)
}

/// A segment shares its start endpoint with itself.
theorem point2_segment_shares_endpoint_self[T: OrderedField](s: Point2Segment[T]) {
    s.shares_endpoint(s)
}

/// Equal first endpoints give a shared endpoint.
theorem point2_segment_shares_endpoint_start_start[T: OrderedField](s: Point2Segment[T], t: Point2Segment[T]) {
    s.start = t.start implies s.shares_endpoint(t)
}

/// The left start endpoint can equal the right end endpoint.
theorem point2_segment_shares_endpoint_start_end[T: OrderedField](s: Point2Segment[T], t: Point2Segment[T]) {
    s.start = t.end implies s.shares_endpoint(t)
}

/// The left end endpoint can equal the right start endpoint.
theorem point2_segment_shares_endpoint_end_start[T: OrderedField](s: Point2Segment[T], t: Point2Segment[T]) {
    s.end = t.start implies s.shares_endpoint(t)
}

/// Equal second endpoints give a shared endpoint.
theorem point2_segment_shares_endpoint_end_end[T: OrderedField](s: Point2Segment[T], t: Point2Segment[T]) {
    s.end = t.end implies s.shares_endpoint(t)
}

/// Endpoint sharing is symmetric.
theorem point2_segment_shares_endpoint_comm[T: OrderedField](s: Point2Segment[T], t: Point2Segment[T]) {
    s.shares_endpoint(t) = t.shares_endpoint(s)
} by {
    if s.shares_endpoint(t) {
        if s.start = t.start {
            t.shares_endpoint(s)
        } else {
            if s.start = t.end {
                t.shares_endpoint(s)
            } else {
                if s.end = t.start {
                    t.shares_endpoint(s)
                } else {
                    s.end = t.end
                    t.shares_endpoint(s)
                }
            }
        }
    }
    if t.shares_endpoint(s) {
        if t.start = s.start {
            s.shares_endpoint(t)
        } else {
            if t.start = s.end {
                s.shares_endpoint(t)
            } else {
                if t.end = s.start {
                    s.shares_endpoint(t)
                } else {
                    t.end = s.end
                    s.shares_endpoint(t)
                }
            }
        }
    }
}

/// Reversing the left segment preserves endpoint sharing in the forward direction.
theorem point2_segment_shares_endpoint_reverse_left_forward[T: OrderedField](s: Point2Segment[T], t: Point2Segment[T]) {
    s.shares_endpoint(t) implies s.reverse.shares_endpoint(t)
} by {
    s.reverse.start = s.end
    s.reverse.end = s.start
    if s.shares_endpoint(t) {
        if s.start = t.start {
            s.reverse.shares_endpoint(t)
        } else {
            if s.start = t.end {
                s.reverse.shares_endpoint(t)
            } else {
                if s.end = t.start {
                    s.reverse.shares_endpoint(t)
                } else {
                    s.end = t.end
                    s.reverse.shares_endpoint(t)
                }
            }
        }
    }
}

/// Reversing the left segment preserves endpoint sharing.
theorem point2_segment_shares_endpoint_reverse_left[T: OrderedField](s: Point2Segment[T], t: Point2Segment[T]) {
    s.reverse.shares_endpoint(t) = s.shares_endpoint(t)
} by {
    if s.reverse.shares_endpoint(t) {
        point2_segment_shares_endpoint_reverse_left_forward(s.reverse, t)
        point2_segment_reverse_reverse(s)
        s.shares_endpoint(t)
    }
    if s.shares_endpoint(t) {
        point2_segment_shares_endpoint_reverse_left_forward(s, t)
    }
}

/// Reversing the right segment preserves endpoint sharing.
theorem point2_segment_shares_endpoint_reverse_right[T: OrderedField](s: Point2Segment[T], t: Point2Segment[T]) {
    s.shares_endpoint(t.reverse) = s.shares_endpoint(t)
} by {
    point2_segment_shares_endpoint_comm(s, t.reverse)
    point2_segment_shares_endpoint_reverse_left(t, s)
    point2_segment_shares_endpoint_comm(t, s)
}

/// Translating two segments preserves endpoint sharing in the forward direction.
theorem point2_segment_shares_endpoint_translate_forward[T: OrderedField](s: Point2Segment[T], t: Point2Segment[T], v: Point2[T]) {
    s.shares_endpoint(t) implies s.translate(v).shares_endpoint(t.translate(v))
} by {
    if s.shares_endpoint(t) {
        if s.start = t.start {
            s.translate(v).start = t.translate(v).start
            s.translate(v).shares_endpoint(t.translate(v))
        } else {
            if s.start = t.end {
                s.translate(v).start = t.translate(v).end
                s.translate(v).shares_endpoint(t.translate(v))
            } else {
                if s.end = t.start {
                    s.translate(v).end = t.translate(v).start
                    s.translate(v).shares_endpoint(t.translate(v))
                } else {
                    s.end = t.end
                    s.translate(v).end = t.translate(v).end
                    s.translate(v).shares_endpoint(t.translate(v))
                }
            }
        }
    }
}

/// Translating two segments preserves endpoint sharing.
theorem point2_segment_shares_endpoint_translate[T: OrderedField](s: Point2Segment[T], t: Point2Segment[T], v: Point2[T]) {
    s.translate(v).shares_endpoint(t.translate(v)) = s.shares_endpoint(t)
} by {
    if s.shares_endpoint(t) {
        point2_segment_shares_endpoint_translate_forward(s, t, v)
    }
    if s.translate(v).shares_endpoint(t.translate(v)) {
        point2_segment_shares_endpoint_translate_forward(s.translate(v), t.translate(v), v.neg)
        point2_segment_translate_neg_right(s, v)
        point2_segment_translate_neg_right(t, v)
        s.shares_endpoint(t)
    }
}

/// Equality of translated starts reflects equality of starts.
theorem point2_segment_translate_start_cancel[T: OrderedField](s: Point2Segment[T], t: Point2Segment[T], v: Point2[T]) {
    s.translate(v).start = t.translate(v).start implies s.start = t.start
} by {
    if s.translate(v).start = t.translate(v).start {
        t.translate(v).start = t.start.translate(v)
        point2_translate_cancel(s.start, t.start, v)
    }
}

/// Equality of translated ends reflects equality of ends.
theorem point2_segment_translate_end_cancel[T: OrderedField](s: Point2Segment[T], t: Point2Segment[T], v: Point2[T]) {
    s.translate(v).end = t.translate(v).end implies s.end = t.end
} by {
    if s.translate(v).end = t.translate(v).end {
        t.translate(v).end = t.end.translate(v)
        point2_translate_cancel(s.end, t.end, v)
    }
}

/// Reversing a segment preserves containment.
theorem point2_segment_reverse_contains[T: OrderedField](s: Point2Segment[T], p: Point2[T]) {
    s.reverse.contains(p) = s.contains(p)
} by {
    point2_on_segment_swap(s.start, s.end, p)
}

/// Translating a contained point gives a contained point in the translated segment.
theorem point2_segment_contains_translate_forward[T: OrderedField](s: Point2Segment[T], p: Point2[T], v: Point2[T]) {
    s.contains(p) implies s.translate(v).contains(p.translate(v))
} by {
    if s.contains(p) {
        point2_on_segment_translate_forward(s.start, s.end, p, v)
        s.start.translate(v).on_segment(s.end.translate(v), p.translate(v))
        s.translate(v).contains(p.translate(v))
    }
}

/// Translating a segment and point preserves containment.
theorem point2_segment_contains_translate[T: OrderedField](s: Point2Segment[T], p: Point2[T], v: Point2[T]) {
    s.translate(v).contains(p.translate(v)) = s.contains(p)
} by {
    point2_on_segment_translate(s.start, s.end, p, v)
}

/// A common contained point gives an intersection.
theorem point2_segments_intersect_of_common_point[T: OrderedField](s: Point2Segment[T], t: Point2Segment[T], p: Point2[T]) {
    s.contains(p) and t.contains(p) implies s.intersects(t)
} by {
    if s.contains(p) and t.contains(p) {
        exists(q: Point2[T]) {
            s.contains(q) and t.contains(q)
        }
    }
}

/// A segment intersects itself.
theorem point2_segments_intersect_self[T: OrderedField](s: Point2Segment[T]) {
    s.intersects(s)
} by {
    point2_segment_contains_start(s)
    point2_segments_intersect_of_common_point(s, s, s.start)
}

/// Segments with equal first endpoints intersect.
theorem point2_segments_intersect_of_start_eq_start[T: OrderedField](s: Point2Segment[T], t: Point2Segment[T]) {
    s.start = t.start implies s.intersects(t)
} by {
    if s.start = t.start {
        point2_segment_contains_start(s)
        point2_segment_contains_start(t)
        t.contains(s.start)
        point2_segments_intersect_of_common_point(s, t, s.start)
    }
}

/// A left start endpoint equal to a right end endpoint gives an intersection.
theorem point2_segments_intersect_of_start_eq_end[T: OrderedField](s: Point2Segment[T], t: Point2Segment[T]) {
    s.start = t.end implies s.intersects(t)
} by {
    if s.start = t.end {
        point2_segment_contains_start(s)
        point2_segment_contains_end(t)
        t.contains(s.start)
        point2_segments_intersect_of_common_point(s, t, s.start)
    }
}

/// A left end endpoint equal to a right start endpoint gives an intersection.
theorem point2_segments_intersect_of_end_eq_start[T: OrderedField](s: Point2Segment[T], t: Point2Segment[T]) {
    s.end = t.start implies s.intersects(t)
} by {
    if s.end = t.start {
        point2_segment_contains_end(s)
        point2_segment_contains_start(t)
        t.contains(s.end)
        point2_segments_intersect_of_common_point(s, t, s.end)
    }
}

/// Segments with equal second endpoints intersect.
theorem point2_segments_intersect_of_end_eq_end[T: OrderedField](s: Point2Segment[T], t: Point2Segment[T]) {
    s.end = t.end implies s.intersects(t)
} by {
    if s.end = t.end {
        point2_segment_contains_end(s)
        point2_segment_contains_end(t)
        t.contains(s.end)
        point2_segments_intersect_of_common_point(s, t, s.end)
    }
}

/// Segments with a shared endpoint intersect.
theorem point2_segments_intersect_of_shared_endpoint[T: OrderedField](s: Point2Segment[T], t: Point2Segment[T]) {
    s.shares_endpoint(t) implies s.intersects(t)
} by {
    let a = s.start = t.start
    let b = s.start = t.end
    let c = s.end = t.start
    let d = s.end = t.end
    let i = s.intersects(t)
    s.shares_endpoint(t) = (((a or b) or c) or d)
    point2_segments_intersect_of_start_eq_start(s, t)
    point2_segments_intersect_of_start_eq_end(s, t)
    point2_segments_intersect_of_end_eq_start(s, t)
    point2_segments_intersect_of_end_eq_end(s, t)
    or_implies(a, b, i)
    or_implies(a or b, c, i)
    or_implies((a or b) or c, d, i)
}

/// Segment intersection is symmetric.
theorem point2_segments_intersect_comm[T: OrderedField](s: Point2Segment[T], t: Point2Segment[T]) {
    s.intersects(t) = t.intersects(s)
} by {
    if s.intersects(t) {
        let p: Point2[T] satisfy {
            s.contains(p) and t.contains(p)
        }
        t.intersects(s)
    }
    if t.intersects(s) {
        let p: Point2[T] satisfy {
            t.contains(p) and s.contains(p)
        }
        s.intersects(t)
    }
}

/// Reversing the left segment preserves intersection.
theorem point2_segments_intersect_reverse_left[T: OrderedField](s: Point2Segment[T], t: Point2Segment[T]) {
    s.reverse.intersects(t) = s.intersects(t)
} by {
    if s.reverse.intersects(t) {
        let p: Point2[T] satisfy {
            s.reverse.contains(p) and t.contains(p)
        }
        point2_segment_reverse_contains(s, p)
        s.intersects(t)
    }
    if s.intersects(t) {
        let p: Point2[T] satisfy {
            s.contains(p) and t.contains(p)
        }
        point2_segment_reverse_contains(s, p)
        s.reverse.intersects(t)
    }
}

/// Reversing the right segment preserves intersection.
theorem point2_segments_intersect_reverse_right[T: OrderedField](s: Point2Segment[T], t: Point2Segment[T]) {
    s.intersects(t.reverse) = s.intersects(t)
} by {
    point2_segments_intersect_comm(s, t.reverse)
    point2_segments_intersect_reverse_left(t, s)
    point2_segments_intersect_comm(t, s)
}

/// Translating intersecting segments gives intersecting translated segments.
theorem point2_segments_intersect_translate_forward[T: OrderedField](s: Point2Segment[T], t: Point2Segment[T], v: Point2[T]) {
    s.intersects(t) implies s.translate(v).intersects(t.translate(v))
} by {
    if s.intersects(t) {
        let p: Point2[T] satisfy {
            s.contains(p) and t.contains(p)
        }
        point2_segment_contains_translate_forward(s, p, v)
        point2_segment_contains_translate_forward(t, p, v)
        exists(q: Point2[T]) {
            s.translate(v).contains(q) and t.translate(v).contains(q)
        }
        s.translate(v).intersects(t.translate(v))
    }
}

/// Translating both segments preserves intersection.
theorem point2_segments_intersect_translate[T: OrderedField](s: Point2Segment[T], t: Point2Segment[T], v: Point2[T]) {
    s.translate(v).intersects(t.translate(v)) = s.intersects(t)
} by {
    if s.intersects(t) {
        point2_segments_intersect_translate_forward(s, t, v)
    }
    if s.translate(v).intersects(t.translate(v)) {
        point2_segments_intersect_translate_forward(s.translate(v), t.translate(v), v.neg)
        point2_segment_translate_neg_right(s, v)
        point2_segment_translate_neg_right(t, v)
        s.intersects(t)
    }
}

/// Translating both segments preserves proper intersection.
theorem point2_segments_properly_intersect_translate[T: OrderedField](s: Point2Segment[T], t: Point2Segment[T], v: Point2[T]) {
    s.translate(v).properly_intersects(t.translate(v)) = s.properly_intersects(t)
} by {
    s.translate(v).start = s.start.translate(v)
    s.translate(v).end = s.end.translate(v)
    t.translate(v).start = t.start.translate(v)
    t.translate(v).end = t.end.translate(v)
    point2_opposite_strict_sides_translate(s.start, s.end, t.start, t.end, v)
    point2_opposite_strict_sides_translate(t.start, t.end, s.start, s.end, v)
}

/// Proper intersection is symmetric.
theorem point2_segments_properly_intersect_comm[T: OrderedField](s: Point2Segment[T], t: Point2Segment[T]) {
    s.properly_intersects(t) = t.properly_intersects(s)
} by {
    if s.properly_intersects(t) {
        t.properly_intersects(s)
    }
    if t.properly_intersects(s) {
        s.properly_intersects(t)
    }
}

/// Reversing the left segment preserves proper intersection.
theorem point2_segments_properly_intersect_reverse_left[T: OrderedField](s: Point2Segment[T], t: Point2Segment[T]) {
    s.reverse.properly_intersects(t) = s.properly_intersects(t)
} by {
    point2_opposite_strict_sides_reverse_line(s.start, s.end, t.start, t.end)
    point2_opposite_strict_sides_swap_points(t.start, t.end, s.start, s.end)
}

/// Reversing the right segment preserves proper intersection.
theorem point2_segments_properly_intersect_reverse_right[T: OrderedField](s: Point2Segment[T], t: Point2Segment[T]) {
    s.properly_intersects(t.reverse) = s.properly_intersects(t)
} by {
    point2_segments_properly_intersect_comm(s, t.reverse)
    point2_segments_properly_intersect_reverse_left(t, s)
    point2_segments_properly_intersect_comm(t, s)
}

/// Reversing both segments preserves proper intersection.
theorem point2_segments_properly_intersect_reverse_both[T: OrderedField](s: Point2Segment[T], t: Point2Segment[T]) {
    s.reverse.properly_intersects(t.reverse) = s.properly_intersects(t)
} by {
    point2_segments_properly_intersect_reverse_left(s, t.reverse)
    point2_segments_properly_intersect_reverse_right(s, t)
}
