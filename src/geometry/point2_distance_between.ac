from nat import Nat
from pair import Pair
from ordered_field import OrderedField
from finite_set import FiniteSet, fs_image, finite_set_image_contains_eq, finite_set_ext,
    finite_set_subset_contains
from data.finite.finite_set_card import fs_card
from data.finite.finite_set_card_compare import fs_card_mono
from data.finite.finite_set_image_card import fs_card_image_le
from data.finite.finite_set_product_card import fs_card_product
from data.finite.finite_set_product import finite_set_product, finite_set_product_contains_eq,
    finite_set_product_contains_pair
from data.finite.finite_set_subset_intro import fs_subset_eq_intro
from geometry.point2 import Point2
from geometry.point2_metric import point2_dist_sq_symmetric

numerals Nat

/// The squared distance between the two points of a pair.
///
/// Named rather than written inline so it can appear in the image below, the same reason
/// `dist_sq_from` is named.
define pair_dist_sq[T: OrderedField](p: Pair[Point2[T], Point2[T]]) -> T {
    p.second.dist_sq(p.first)
}

/// The set of squared distances realized between two finite planar sets.
///
/// The distance set of a single point generalises to a pair of sets by taking the image of
/// the product, which is where a bipartite distance count has to start: the pairs are the
/// objects being counted, and `distances_from` only ever ranges over one endpoint.
define distances_between[T: OrderedField](
    a: FiniteSet[Point2[T]], b: FiniteSet[Point2[T]]
) -> FiniteSet[T] {
    fs_image(finite_set_product(a, b), pair_dist_sq[T])
}

/// Membership in the bipartite distance set.
theorem distances_between_contains_eq[T: OrderedField](
    a: FiniteSet[Point2[T]], b: FiniteSet[Point2[T]], t: T
) {
    distances_between(a, b).contains(t) = exists(p: Pair[Point2[T], Point2[T]]) {
        finite_set_product(a, b).contains(p) and t = pair_dist_sq(p)
    }
} by {
    finite_set_image_contains_eq(finite_set_product(a, b), pair_dist_sq[T], t)
}

/// A pair of members realizes its distance.
theorem distances_between_contains[T: OrderedField](
    a: FiniteSet[Point2[T]], b: FiniteSet[Point2[T]], x: Point2[T], y: Point2[T]
) {
    a.contains(x) and b.contains(y) implies distances_between(a, b).contains(y.dist_sq(x))
} by {
    if a.contains(x) and b.contains(y) {
        finite_set_product_contains_pair(a, b, x, y)
        finite_set_product(a, b).contains(Pair.new(x, y))
        Pair.new(x, y).first = x
        Pair.new(x, y).second = y
        pair_dist_sq(Pair.new(x, y)) = y.dist_sq(x)
        exists(p: Pair[Point2[T], Point2[T]]) {
            finite_set_product(a, b).contains(p) and y.dist_sq(x) = pair_dist_sq(p)
        }
        distances_between_contains_eq(a, b, y.dist_sq(x))
        distances_between(a, b).contains(y.dist_sq(x))
    }
}

/// Every realized distance comes from a pair of members.
theorem distances_between_witness[T: OrderedField](
    a: FiniteSet[Point2[T]], b: FiniteSet[Point2[T]], t: T
) {
    distances_between(a, b).contains(t) implies exists(x: Point2[T], y: Point2[T]) {
        a.contains(x) and b.contains(y) and t = y.dist_sq(x)
    }
} by {
    if distances_between(a, b).contains(t) {
        distances_between_contains_eq(a, b, t)
        exists(p: Pair[Point2[T], Point2[T]]) {
            finite_set_product(a, b).contains(p) and t = pair_dist_sq(p)
        }
        let (p: Pair[Point2[T], Point2[T]]) satisfy {
            finite_set_product(a, b).contains(p) and t = pair_dist_sq(p)
        }
        finite_set_product_contains_eq(a, b, p)
        a.contains(p.first) and b.contains(p.second)
        t = p.second.dist_sq(p.first)
        exists(x: Point2[T], y: Point2[T]) {
            a.contains(x) and b.contains(y) and t = y.dist_sq(x)
        }
    }
}

/// The bipartite distance set is symmetric in its two arguments.
///
/// The squared distance does not depend on which endpoint is named first, so the two images
/// have the same members.
theorem distances_between_symmetric[T: OrderedField](
    a: FiniteSet[Point2[T]], b: FiniteSet[Point2[T]]
) {
    distances_between(a, b) = distances_between(b, a)
} by {
    forall(t: T) {
        if distances_between(a, b).contains(t) {
            distances_between_witness(a, b, t)
            let (x: Point2[T], y: Point2[T]) satisfy {
                a.contains(x) and b.contains(y) and t = y.dist_sq(x)
            }
            point2_dist_sq_symmetric(y, x)
            y.dist_sq(x) = x.dist_sq(y)
            t = x.dist_sq(y)
            distances_between_contains(b, a, y, x)
            distances_between(b, a).contains(x.dist_sq(y))
            distances_between(b, a).contains(t)
        }
        if distances_between(b, a).contains(t) {
            distances_between_witness(b, a, t)
            let (x: Point2[T], y: Point2[T]) satisfy {
                b.contains(x) and a.contains(y) and t = y.dist_sq(x)
            }
            point2_dist_sq_symmetric(y, x)
            y.dist_sq(x) = x.dist_sq(y)
            t = x.dist_sq(y)
            distances_between_contains(a, b, y, x)
            distances_between(a, b).contains(x.dist_sq(y))
            distances_between(a, b).contains(t)
        }
        (distances_between(a, b).contains(t) implies distances_between(b, a).contains(t))
        (distances_between(b, a).contains(t) implies distances_between(a, b).contains(t))
        distances_between(a, b).contains(t) = distances_between(b, a).contains(t)
    }
    finite_set_ext(distances_between(a, b), distances_between(b, a))
}

/// Enlarging either point set can only add distances.
theorem distances_between_mono[T: OrderedField](
    a: FiniteSet[Point2[T]], b: FiniteSet[Point2[T]],
    c: FiniteSet[Point2[T]], d: FiniteSet[Point2[T]]
) {
    a.subset_eq(c) and b.subset_eq(d)
        implies distances_between(a, b).subset_eq(distances_between(c, d))
} by {
    if a.subset_eq(c) and b.subset_eq(d) {
        forall(t: T) {
            if distances_between(a, b).contains(t) {
                distances_between_witness(a, b, t)
                let (x: Point2[T], y: Point2[T]) satisfy {
                    a.contains(x) and b.contains(y) and t = y.dist_sq(x)
                }
                finite_set_subset_contains(a, c, x)
                c.contains(x)
                finite_set_subset_contains(b, d, y)
                d.contains(y)
                distances_between_contains(c, d, x, y)
                distances_between(c, d).contains(y.dist_sq(x))
                distances_between(c, d).contains(t)
            }
            (distances_between(a, b).contains(t)
                implies distances_between(c, d).contains(t))
        }
        fs_subset_eq_intro(distances_between(a, b), distances_between(c, d))
        distances_between(a, b).subset_eq(distances_between(c, d))
    }
}

/// The number of squared distances realized between two finite planar sets.
define distance_count_between[T: OrderedField](
    a: FiniteSet[Point2[T]], b: FiniteSet[Point2[T]]
) -> Nat {
    fs_card(distances_between(a, b))
}

/// The bipartite distance count is monotone in both point sets.
theorem distance_count_between_mono[T: OrderedField](
    a: FiniteSet[Point2[T]], b: FiniteSet[Point2[T]],
    c: FiniteSet[Point2[T]], d: FiniteSet[Point2[T]]
) {
    a.subset_eq(c) and b.subset_eq(d)
        implies distance_count_between(a, b) <= distance_count_between(c, d)
} by {
    if a.subset_eq(c) and b.subset_eq(d) {
        distances_between_mono(a, b, c, d)
        distances_between(a, b).subset_eq(distances_between(c, d))
        fs_card_mono(distances_between(a, b), distances_between(c, d))
        fs_card(distances_between(a, b)) <= fs_card(distances_between(c, d))
        distance_count_between(a, b) <= distance_count_between(c, d)
    }
}

/// No more distances are realized than there are pairs of points.
///
/// The bipartite analogue of `distance_count_le_card`: the distance set is the image of the
/// product, and an image is no larger than its domain.
theorem distance_count_between_le_card[T: OrderedField](
    a: FiniteSet[Point2[T]], b: FiniteSet[Point2[T]]
) {
    distance_count_between(a, b) <= fs_card(b) * fs_card(a)
} by {
    fs_card_image_le(finite_set_product(a, b), pair_dist_sq[T])
    (fs_card(fs_image(finite_set_product(a, b), pair_dist_sq[T]))
        <= fs_card(finite_set_product(a, b)))
    fs_card(distances_between(a, b)) <= fs_card(finite_set_product(a, b))
    fs_card_product(a, b)
    fs_card(finite_set_product(a, b)) = fs_card(b) * fs_card(a)
    distance_count_between(a, b) <= fs_card(b) * fs_card(a)
}
