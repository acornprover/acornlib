from ordered_field import OrderedField
from geometry.point2 import Point2
from geometry.point2_algebra import point2_norm_sq_nonneg,
    point2_norm_sq_neg, point2_dist_sq_self, point2_dist_sq_comm,
    point2_dist_sq_eq_norm_sq_sub, point2_sub_reverse_neg
from geometry.point2_affine import point2_sub_translate, point2_dist_sq_translate

/// Squared coordinate distance is nonnegative.
theorem point2_dist_sq_nonneg[T: OrderedField](a: Point2[T], b: Point2[T]) {
    a.dist_sq(b) >= T.0
} by {
    point2_norm_sq_nonneg(a.sub(b))
}

/// The squared distance from a point to itself is zero.
theorem point2_dist_sq_zero_self[T: OrderedField](a: Point2[T]) {
    a.dist_sq(a) = T.0
} by {
    point2_dist_sq_self(a)
}

/// Squared distance is symmetric.
theorem point2_dist_sq_symmetric[T: OrderedField](a: Point2[T], b: Point2[T]) {
    a.dist_sq(b) = b.dist_sq(a)
} by {
    point2_dist_sq_comm(a, b)
}

/// Squared distance is the squared norm of the displacement vector.
theorem point2_dist_sq_as_norm_sq_sub[T: OrderedField](a: Point2[T], b: Point2[T]) {
    a.dist_sq(b) = a.sub(b).norm_sq
} by {
    point2_dist_sq_eq_norm_sq_sub(a, b)
}

/// Squared distance may use the displacement in either direction.
theorem point2_dist_sq_eq_norm_sq_sub_reverse[T: OrderedField](a: Point2[T], b: Point2[T]) {
    a.dist_sq(b) = b.sub(a).norm_sq
} by {
    point2_sub_reverse_neg(a, b)
    point2_norm_sq_neg(b.sub(a))
}

/// Translating both points preserves squared distance.
theorem point2_dist_sq_translate_invariant[T: OrderedField](a: Point2[T], b: Point2[T], v: Point2[T]) {
    a.translate(v).dist_sq(b.translate(v)) = a.dist_sq(b)
} by {
    point2_dist_sq_translate(a, b, v)
}

/// Translating both points preserves squared-distance nonnegativity.
theorem point2_dist_sq_translate_nonneg[T: OrderedField](a: Point2[T], b: Point2[T], v: Point2[T]) {
    a.translate(v).dist_sq(b.translate(v)) >= T.0
} by {
    point2_dist_sq_nonneg(a.translate(v), b.translate(v))
}

/// A translated displacement has the same squared norm as the original displacement.
theorem point2_norm_sq_sub_translate[T: OrderedField](a: Point2[T], b: Point2[T], v: Point2[T]) {
    a.translate(v).sub(b.translate(v)).norm_sq = a.sub(b).norm_sq
} by {
    point2_sub_translate(a, b, v)
}

/// Squared distance from a translated point to a translated center equals the original squared distance.
theorem point2_dist_sq_translate_center[T: OrderedField](a: Point2[T], center: Point2[T], v: Point2[T]) {
    a.translate(v).dist_sq(center.translate(v)) = a.dist_sq(center)
} by {
    point2_dist_sq_translate(a, center, v)
}

/// Squared distance from a point to a center equals the reversed center-to-point squared distance.
theorem point2_dist_sq_center_symm[T: OrderedField](a: Point2[T], center: Point2[T]) {
    a.dist_sq(center) = center.dist_sq(a)
} by {
    point2_dist_sq_comm(a, center)
}

/// Equal points have zero squared distance.
theorem point2_dist_sq_eq_zero_of_eq[T: OrderedField](a: Point2[T], b: Point2[T]) {
    a = b implies a.dist_sq(b) = T.0
} by {
    if a = b {
        point2_dist_sq_self(b)
    }
}

/// A point has nonnegative squared distance to itself.
theorem point2_dist_sq_self_nonneg[T: OrderedField](a: Point2[T]) {
    a.dist_sq(a) >= T.0
} by {
    point2_dist_sq_nonneg(a, a)
}

/// Symmetric squared distance is nonnegative in either order.
theorem point2_dist_sq_comm_nonneg[T: OrderedField](a: Point2[T], b: Point2[T]) {
    b.dist_sq(a) >= T.0
} by {
    point2_dist_sq_nonneg(b, a)
}

/// Translating equal points gives zero squared distance.
theorem point2_dist_sq_translate_eq_zero_of_eq[T: OrderedField](a: Point2[T], b: Point2[T], v: Point2[T]) {
    a = b implies a.translate(v).dist_sq(b.translate(v)) = T.0
} by {
    if a = b {
        point2_dist_sq_translate(a, b, v)
        point2_dist_sq_eq_zero_of_eq(a, b)
    }
}

/// Squared distance to a translated center is nonnegative.
theorem point2_dist_sq_translate_center_nonneg[T: OrderedField](a: Point2[T], center: Point2[T], v: Point2[T]) {
    a.translate(v).dist_sq(center.translate(v)) >= T.0
} by {
    point2_dist_sq_nonneg(a.translate(v), center.translate(v))
}

/// Reversing the arguments preserves squared-distance nonnegativity.
theorem point2_dist_sq_reverse_nonneg[T: OrderedField](a: Point2[T], b: Point2[T]) {
    a.dist_sq(b) >= T.0 and b.dist_sq(a) >= T.0
} by {
    point2_dist_sq_nonneg(a, b)
    point2_dist_sq_nonneg(b, a)
}
