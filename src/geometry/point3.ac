from algebra.add_comm_monoid import AddCommMonoid
from algebra.add_comm_group import AddCommGroup
from comm_ring import CommRing
from algebra.ring.ring import mul_zero_left, mul_zero_right
from ordered_field import OrderedField

/// A three-dimensional point with coordinates in `T`.
structure Point3[T] {
    /// The x-coordinate.
    x: T

    /// The y-coordinate.
    y: T

    /// The z-coordinate.
    z: T
}

/// The origin in a coordinate space with additive identity.
let point3_zero[T: AddCommMonoid]: Point3[T] = Point3.new(T.0, T.0, T.0)

/// Rebuilding a point from its coordinates gives the original point.
theorem point3_eta[T](p: Point3[T]) {
    Point3.new(p.x, p.y, p.z) = p
}

/// A point is determined by its three coordinates.
theorem point3_ext[T](p: Point3[T], q: Point3[T]) {
    p.x = q.x and p.y = q.y and p.z = q.z implies p = q
} by {
    if p.x = q.x and p.y = q.y and p.z = q.z {
        point3_eta(p)
        Point3.new(p.x, p.y, p.z) = p
        point3_eta(q)
        Point3.new(q.x, q.y, q.z) = q
        p = q
    }
}

/// The x-coordinate of a newly constructed point.
theorem point3_new_x[T](x: T, y: T, z: T) {
    Point3.new(x, y, z).x = x
}

/// The y-coordinate of a newly constructed point.
theorem point3_new_y[T](x: T, y: T, z: T) {
    Point3.new(x, y, z).y = y
}

/// The z-coordinate of a newly constructed point.
theorem point3_new_z[T](x: T, y: T, z: T) {
    Point3.new(x, y, z).z = z
}

attributes Point3[T: AddCommMonoid] {
    /// The origin of the coordinate space.
    let zero: Point3[T] = point3_zero[T]

    /// The componentwise sum of two points.
    define add(self, other: Point3[T]) -> Point3[T] {
        Point3.new(self.x + other.x, self.y + other.y, self.z + other.z)
    }
}

/// The x-coordinate of the origin.
theorem point3_zero_x[T: AddCommMonoid] {
    point3_zero[T].x = T.0
}

/// The y-coordinate of the origin.
theorem point3_zero_y[T: AddCommMonoid] {
    point3_zero[T].y = T.0
}

/// The z-coordinate of the origin.
theorem point3_zero_z[T: AddCommMonoid] {
    point3_zero[T].z = T.0
}

/// The x-coordinate of a point sum.
theorem point3_add_x[T: AddCommMonoid](p: Point3[T], q: Point3[T]) {
    p.add(q).x = p.x + q.x
}

/// The y-coordinate of a point sum.
theorem point3_add_y[T: AddCommMonoid](p: Point3[T], q: Point3[T]) {
    p.add(q).y = p.y + q.y
}

/// The z-coordinate of a point sum.
theorem point3_add_z[T: AddCommMonoid](p: Point3[T], q: Point3[T]) {
    p.add(q).z = p.z + q.z
}

/// Point addition is commutative.
theorem point3_add_comm[T: AddCommMonoid](p: Point3[T], q: Point3[T]) {
    p.add(q) = q.add(p)
} by {
    let lhs = p.add(q)
    let rhs = q.add(p)
    rhs.x = q.x + p.x
    rhs.y = q.y + p.y
    rhs.z = q.z + p.z
    point3_ext(lhs, rhs)
}

/// Point addition is associative.
theorem point3_add_assoc[T: AddCommMonoid](p: Point3[T], q: Point3[T], r: Point3[T]) {
    p.add(q).add(r) = p.add(q.add(r))
} by {
    let lhs = p.add(q).add(r)
    let rhs = p.add(q.add(r))
    lhs.x = (p.x + q.x) + r.x
    rhs.x = p.x + (q.x + r.x)
    lhs.x = rhs.x
    lhs.y = (p.y + q.y) + r.y
    rhs.y = p.y + (q.y + r.y)
    lhs.y = rhs.y
    lhs.z = (p.z + q.z) + r.z
    rhs.z = p.z + (q.z + r.z)
    lhs.z = rhs.z
    point3_ext(lhs, rhs)
}

/// Adding the origin on the right leaves a point unchanged.
theorem point3_add_zero_right[T: AddCommMonoid](p: Point3[T]) {
    p.add(point3_zero[T]) = p
} by {
    let lhs = p.add(point3_zero[T])
    point3_ext(lhs, p)
}

/// Adding the origin on the left leaves a point unchanged.
theorem point3_add_zero_left[T: AddCommMonoid](p: Point3[T]) {
    point3_zero[T].add(p) = p
} by {
    let lhs = point3_zero[T].add(p)
    point3_ext(lhs, p)
}

attributes Point3[T: AddCommGroup] {
    /// The componentwise additive inverse of a point.
    define neg(self) -> Point3[T] {
        Point3.new(-self.x, -self.y, -self.z)
    }

    /// The componentwise difference of two points.
    define sub(self, other: Point3[T]) -> Point3[T] {
        Point3.new(self.x - other.x, self.y - other.y, self.z - other.z)
    }

    /// Translation by a coordinate displacement.
    define translate(self, displacement: Point3[T]) -> Point3[T] {
        self.add(displacement)
    }
}

/// The x-coordinate of a point inverse.
theorem point3_neg_x[T: AddCommGroup](p: Point3[T]) {
    p.neg.x = -p.x
}

/// The y-coordinate of a point inverse.
theorem point3_neg_y[T: AddCommGroup](p: Point3[T]) {
    p.neg.y = -p.y
}

/// The z-coordinate of a point inverse.
theorem point3_neg_z[T: AddCommGroup](p: Point3[T]) {
    p.neg.z = -p.z
}

/// The x-coordinate of a point difference.
theorem point3_sub_x[T: AddCommGroup](p: Point3[T], q: Point3[T]) {
    p.sub(q).x = p.x - q.x
}

/// The y-coordinate of a point difference.
theorem point3_sub_y[T: AddCommGroup](p: Point3[T], q: Point3[T]) {
    p.sub(q).y = p.y - q.y
}

/// The z-coordinate of a point difference.
theorem point3_sub_z[T: AddCommGroup](p: Point3[T], q: Point3[T]) {
    p.sub(q).z = p.z - q.z
}

/// The x-coordinate of a translated point.
theorem point3_translate_x[T: AddCommGroup](p: Point3[T], displacement: Point3[T]) {
    p.translate(displacement).x = p.x + displacement.x
}

/// The y-coordinate of a translated point.
theorem point3_translate_y[T: AddCommGroup](p: Point3[T], displacement: Point3[T]) {
    p.translate(displacement).y = p.y + displacement.y
}

/// The z-coordinate of a translated point.
theorem point3_translate_z[T: AddCommGroup](p: Point3[T], displacement: Point3[T]) {
    p.translate(displacement).z = p.z + displacement.z
}

/// A point minus itself is the origin.
theorem point3_sub_self[T: AddCommGroup](p: Point3[T]) {
    p.sub(p) = point3_zero[T]
} by {
    let lhs = p.sub(p)
    lhs.x = p.x - p.x
    lhs.x = T.0
    lhs.y = p.y - p.y
    lhs.y = T.0
    lhs.z = p.z - p.z
    lhs.z = T.0
    point3_ext(lhs, point3_zero[T])
}

/// Adding a point and its additive inverse gives the origin.
theorem point3_add_neg_right[T: AddCommGroup](p: Point3[T]) {
    p.add(p.neg) = point3_zero[T]
} by {
    let lhs = p.add(p.neg)
    lhs.x = p.x + -p.x
    lhs.x = T.0
    lhs.y = p.y + -p.y
    lhs.y = T.0
    lhs.z = p.z + -p.z
    lhs.z = T.0
    point3_ext(lhs, point3_zero[T])
}

attributes Point3[T: CommRing] {
    /// Scalar multiplication of all three coordinates.
    define smul(self, scalar: T) -> Point3[T] {
        Point3.new(scalar * self.x, scalar * self.y, scalar * self.z)
    }

    /// The dot product of two coordinate points.
    define dot(self, other: Point3[T]) -> T {
        self.x * other.x + self.y * other.y + self.z * other.z
    }

    /// The vector cross product of two coordinate points.
    define cross(self, other: Point3[T]) -> Point3[T] {
        Point3.new(
            self.y * other.z - self.z * other.y,
            self.z * other.x - self.x * other.z,
            self.x * other.y - self.y * other.x)
    }

    /// The squared coordinate norm.
    define norm_sq(self) -> T {
        self.dot(self)
    }

    /// The squared coordinate distance to another point.
    define dist_sq(self, other: Point3[T]) -> T {
        self.sub(other).norm_sq
    }

    /// The point with parameter `t` on the directed line to `other`.
    define param_line(self, other: Point3[T], t: T) -> Point3[T] {
        self.add(other.sub(self).smul(t))
    }
}

/// The x-coordinate of a scalar multiple.
theorem point3_smul_x[T: CommRing](p: Point3[T], scalar: T) {
    p.smul(scalar).x = scalar * p.x
}

/// The y-coordinate of a scalar multiple.
theorem point3_smul_y[T: CommRing](p: Point3[T], scalar: T) {
    p.smul(scalar).y = scalar * p.y
}

/// The z-coordinate of a scalar multiple.
theorem point3_smul_z[T: CommRing](p: Point3[T], scalar: T) {
    p.smul(scalar).z = scalar * p.z
}

/// The dot product is symmetric.
theorem point3_dot_comm[T: CommRing](p: Point3[T], q: Point3[T]) {
    p.dot(q) = q.dot(p)
} by {
}

/// The x-coordinate of a cross product.
theorem point3_cross_x[T: CommRing](a: Point3[T], b: Point3[T]) {
    a.cross(b).x = a.y * b.z - a.z * b.y
}

/// The y-coordinate of a cross product.
theorem point3_cross_y[T: CommRing](a: Point3[T], b: Point3[T]) {
    a.cross(b).y = a.z * b.x - a.x * b.z
}

/// The z-coordinate of a cross product.
theorem point3_cross_z[T: CommRing](a: Point3[T], b: Point3[T]) {
    a.cross(b).z = a.x * b.y - a.y * b.x
}

/// The cross product of a point with itself is zero.
theorem point3_cross_self[T: CommRing](p: Point3[T]) {
    p.cross(p) = point3_zero[T]
} by {
    let lhs = p.cross(p)
    point3_cross_x(p, p)
    lhs.x = p.y * p.z - p.z * p.y
    p.z * p.y = p.y * p.z
    p.y * p.z - p.z * p.y = T.0
    lhs.x = T.0
    point3_cross_y(p, p)
    lhs.y = p.z * p.x - p.x * p.z
    p.x * p.z = p.z * p.x
    p.z * p.x - p.x * p.z = T.0
    lhs.y = T.0
    point3_cross_z(p, p)
    lhs.z = p.x * p.y - p.y * p.x
    p.y * p.x = p.x * p.y
    p.x * p.y - p.y * p.x = T.0
    lhs.z = T.0
    point3_ext(lhs, point3_zero[T])
}

/// The cross product with the origin on the left is zero.
theorem point3_cross_zero_left[T: CommRing](p: Point3[T]) {
    point3_zero[T].cross(p) = point3_zero[T]
} by {
    let lhs = point3_zero[T].cross(p)
    point3_cross_x(point3_zero[T], p)
    lhs.x = T.0 * p.z - T.0 * p.y
    mul_zero_left[T](p.z)
    mul_zero_left[T](p.y)
    T.0 * p.z - T.0 * p.y = T.0 - T.0
    T.0 - T.0 = T.0
    lhs.x = T.0
    point3_cross_y(point3_zero[T], p)
    lhs.y = T.0 * p.x - T.0 * p.z
    mul_zero_left[T](p.x)
    mul_zero_left[T](p.z)
    T.0 * p.x - T.0 * p.z = T.0 - T.0
    T.0 - T.0 = T.0
    lhs.y = T.0
    point3_cross_z(point3_zero[T], p)
    lhs.z = T.0 * p.y - T.0 * p.x
    mul_zero_left[T](p.y)
    mul_zero_left[T](p.x)
    T.0 * p.y - T.0 * p.x = T.0 - T.0
    T.0 - T.0 = T.0
    lhs.z = T.0
    point3_ext(lhs, point3_zero[T])
}

/// The cross product with the origin on the right is zero.
theorem point3_cross_zero_right[T: CommRing](p: Point3[T]) {
    p.cross(point3_zero[T]) = point3_zero[T]
} by {
    let lhs = p.cross(point3_zero[T])
    point3_cross_x(p, point3_zero[T])
    lhs.x = p.y * T.0 - p.z * T.0
    mul_zero_right[T](p.y)
    mul_zero_right[T](p.z)
    p.y * T.0 - p.z * T.0 = T.0 - T.0
    T.0 - T.0 = T.0
    lhs.x = T.0
    point3_cross_y(p, point3_zero[T])
    lhs.y = p.z * T.0 - p.x * T.0
    mul_zero_right[T](p.z)
    mul_zero_right[T](p.x)
    p.z * T.0 - p.x * T.0 = T.0 - T.0
    T.0 - T.0 = T.0
    lhs.y = T.0
    point3_cross_z(p, point3_zero[T])
    lhs.z = p.x * T.0 - p.y * T.0
    mul_zero_right[T](p.x)
    mul_zero_right[T](p.y)
    p.x * T.0 - p.y * T.0 = T.0 - T.0
    T.0 - T.0 = T.0
    lhs.z = T.0
    point3_ext(lhs, point3_zero[T])
}


