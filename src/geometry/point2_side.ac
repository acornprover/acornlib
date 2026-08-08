from ordered_field import OrderedField
from algebra.add_ordered_group import neg_lt_neg, lt_of_neg_lt_neg
from data.basic.logic import and_comm, or_comm
from geometry.point2 import Point2
from geometry.point2_orientation import point2_left_turn_not_collinear,
    point2_right_turn_not_collinear, point2_orientation_swap_first_second_neg
from geometry.point2_affine import point2_left_turn_translate,
    point2_right_turn_translate
from geometry.point2_segment import point2_on_line_iff_collinear

attributes Point2[T: OrderedField] {
    /// True when `p` lies strictly to the left of the directed line from `self` to `b`.
    define left_of_line(self, b: Point2[T], p: Point2[T]) -> Bool {
        self.left_turn(b, p)
    }

    /// True when `p` lies strictly to the right of the directed line from `self` to `b`.
    define right_of_line(self, b: Point2[T], p: Point2[T]) -> Bool {
        self.right_turn(b, p)
    }

    /// True when two points lie strictly on the same side of a directed line.
    define same_strict_side(self, b: Point2[T], p: Point2[T], q: Point2[T]) -> Bool {
        (self.left_of_line(b, p) and self.left_of_line(b, q)) or
        (self.right_of_line(b, p) and self.right_of_line(b, q))
    }

    /// True when two points lie strictly on opposite sides of a directed line.
    define opposite_strict_sides(self, b: Point2[T], p: Point2[T], q: Point2[T]) -> Bool {
        (self.left_of_line(b, p) and self.right_of_line(b, q)) or
        (self.right_of_line(b, p) and self.left_of_line(b, q))
    }
}

/// Strict left-of-line is the left-turn predicate.
theorem point2_left_of_line_eq_left_turn[T: OrderedField](a: Point2[T], b: Point2[T], p: Point2[T]) {
    a.left_of_line(b, p) = a.left_turn(b, p)
}

/// Strict right-of-line is the right-turn predicate.
theorem point2_right_of_line_eq_right_turn[T: OrderedField](a: Point2[T], b: Point2[T], p: Point2[T]) {
    a.right_of_line(b, p) = a.right_turn(b, p)
}

/// Reversing the directed line turns strict left-of-line into strict right-of-line.
theorem point2_left_of_line_reverse[T: OrderedField](a: Point2[T], b: Point2[T], p: Point2[T]) {
    b.left_of_line(a, p) = a.right_of_line(b, p)
} by {
    point2_orientation_swap_first_second_neg(b, a, p)
    if b.left_of_line(a, p) {
        b.orientation(a, p) > T.0
        -T.0 = T.0
        lt_of_neg_lt_neg(a.orientation(b, p), T.0)
        a.orientation(b, p) < T.0
        a.right_turn(b, p)
        a.right_of_line(b, p)
    }
    if a.right_of_line(b, p) {
        neg_lt_neg(a.orientation(b, p), T.0)
        -T.0 < -a.orientation(b, p)
        T.0 < -a.orientation(b, p)
        b.left_turn(a, p)
        b.left_of_line(a, p)
    }
}

/// Reversing the directed line turns strict right-of-line into strict left-of-line.
theorem point2_right_of_line_reverse[T: OrderedField](a: Point2[T], b: Point2[T], p: Point2[T]) {
    b.right_of_line(a, p) = a.left_of_line(b, p)
} by {
    point2_orientation_swap_first_second_neg(b, a, p)
    if b.right_of_line(a, p) {
        b.right_turn(a, p)
        b.orientation(a, p) < T.0
        b.orientation(a, p) = -a.orientation(b, p)
        -a.orientation(b, p) < T.0
        -a.orientation(b, p) < -T.0
        lt_of_neg_lt_neg(T.0, a.orientation(b, p))
        a.left_of_line(b, p)
    }
    if a.left_of_line(b, p) {
        T.0 < a.orientation(b, p)
        neg_lt_neg(T.0, a.orientation(b, p))
        -a.orientation(b, p) < T.0
        b.right_of_line(a, p)
    }
}

/// A point strictly left of a directed line is not on the line.
theorem point2_left_of_line_not_on_line[T: OrderedField](a: Point2[T], b: Point2[T], p: Point2[T]) {
    a.left_of_line(b, p) implies not a.on_line(b, p)
} by {
    if a.left_of_line(b, p) {
        point2_left_turn_not_collinear(a, b, p)
        point2_on_line_iff_collinear(a, b, p)
        not a.on_line(b, p)
    }
}

/// A point strictly right of a directed line is not on the line.
theorem point2_right_of_line_not_on_line[T: OrderedField](a: Point2[T], b: Point2[T], p: Point2[T]) {
    a.right_of_line(b, p) implies not a.on_line(b, p)
} by {
    if a.right_of_line(b, p) {
        point2_right_turn_not_collinear(a, b, p)
        point2_on_line_iff_collinear(a, b, p)
        not a.on_line(b, p)
    }
}

/// Translating all points preserves strict left-of-line.
theorem point2_left_of_line_translate[T: OrderedField](a: Point2[T], b: Point2[T], p: Point2[T], v: Point2[T]) {
    a.translate(v).left_of_line(b.translate(v), p.translate(v)) = a.left_of_line(b, p)
} by {
    point2_left_turn_translate(a, b, p, v)
}

/// Translating all points preserves strict right-of-line.
theorem point2_right_of_line_translate[T: OrderedField](a: Point2[T], b: Point2[T], p: Point2[T], v: Point2[T]) {
    a.translate(v).right_of_line(b.translate(v), p.translate(v)) = a.right_of_line(b, p)
} by {
    point2_right_turn_translate(a, b, p, v)
}

/// Strict same-side is symmetric in the two witness points.
theorem point2_same_strict_side_swap_points[T: OrderedField](a: Point2[T], b: Point2[T], p: Point2[T], q: Point2[T]) {
    a.same_strict_side(b, q, p) = a.same_strict_side(b, p, q)
} by {
    let lp = a.left_of_line(b, p)
    let rp = a.right_of_line(b, p)
    let lq = a.left_of_line(b, q)
    let rq = a.right_of_line(b, q)
    and_comm(lq, lp)
    and_comm(rq, rp)
}

/// Strict opposite-side is symmetric in the two witness points.
theorem point2_opposite_strict_sides_swap_points[T: OrderedField](a: Point2[T], b: Point2[T], p: Point2[T], q: Point2[T]) {
    a.opposite_strict_sides(b, q, p) = a.opposite_strict_sides(b, p, q)
} by {
    let lp = a.left_of_line(b, p)
    let rp = a.right_of_line(b, p)
    let lq = a.left_of_line(b, q)
    let rq = a.right_of_line(b, q)
    and_comm(lq, rp)
    and_comm(rq, lp)
    or_comm(rp and lq, lp and rq)
}

/// Strict same-side is invariant under reversing the directed line.
theorem point2_same_strict_side_reverse_line[T: OrderedField](a: Point2[T], b: Point2[T], p: Point2[T], q: Point2[T]) {
    b.same_strict_side(a, p, q) = a.same_strict_side(b, p, q)
} by {
    point2_left_of_line_reverse(a, b, p)
    point2_left_of_line_reverse(a, b, q)
    point2_right_of_line_reverse(a, b, p)
    point2_right_of_line_reverse(a, b, q)
    let lp = a.left_of_line(b, p)
    let rp = a.right_of_line(b, p)
    let lq = a.left_of_line(b, q)
    let rq = a.right_of_line(b, q)
    or_comm(rp and rq, lp and lq)
}

/// Strict opposite-side is invariant under reversing the directed line.
theorem point2_opposite_strict_sides_reverse_line[T: OrderedField](a: Point2[T], b: Point2[T], p: Point2[T], q: Point2[T]) {
    b.opposite_strict_sides(a, p, q) = a.opposite_strict_sides(b, p, q)
} by {
    point2_left_of_line_reverse(a, b, p)
    point2_left_of_line_reverse(a, b, q)
    point2_right_of_line_reverse(a, b, p)
    point2_right_of_line_reverse(a, b, q)
    let lp = a.left_of_line(b, p)
    let rp = a.right_of_line(b, p)
    let lq = a.left_of_line(b, q)
    let rq = a.right_of_line(b, q)
    or_comm(rp and lq, lp and rq)
}

/// Translating all points preserves strict same-side.
theorem point2_same_strict_side_translate[T: OrderedField](a: Point2[T], b: Point2[T], p: Point2[T], q: Point2[T], v: Point2[T]) {
    a.translate(v).same_strict_side(b.translate(v), p.translate(v), q.translate(v)) =
    a.same_strict_side(b, p, q)
} by {
    point2_left_of_line_translate(a, b, p, v)
    point2_left_of_line_translate(a, b, q, v)
    point2_right_of_line_translate(a, b, p, v)
    point2_right_of_line_translate(a, b, q, v)
}

/// Translating all points preserves strict opposite-side.
theorem point2_opposite_strict_sides_translate[T: OrderedField](a: Point2[T], b: Point2[T], p: Point2[T], q: Point2[T], v: Point2[T]) {
    a.translate(v).opposite_strict_sides(b.translate(v), p.translate(v), q.translate(v)) =
    a.opposite_strict_sides(b, p, q)
} by {
    point2_left_of_line_translate(a, b, p, v)
    point2_left_of_line_translate(a, b, q, v)
    point2_right_of_line_translate(a, b, p, v)
    point2_right_of_line_translate(a, b, q, v)
}
