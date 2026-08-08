from algebra.add_comm_monoid import AddCommMonoid
from algebra.add_comm_group import AddCommGroup
from comm_ring import CommRing
from algebra.ring.ring import mul_zero_left, mul_zero_right
from ordered_field import OrderedField

/// A two-dimensional point with coordinates in `T`.
structure Point2[T] {
    /// The x-coordinate.
    x: T

    /// The y-coordinate.
    y: T
}

/// The origin in a coordinate plane with additive identity.
let point2_zero[T: AddCommMonoid]: Point2[T] = Point2.new(T.0, T.0)

/// A point is determined by its two coordinates.
theorem point2_ext[T](p: Point2[T], q: Point2[T]) {
    p.x = q.x and p.y = q.y implies p = q
}

/// Rebuilding a point from its coordinates gives the original point.
theorem point2_eta[T](p: Point2[T]) {
    Point2.new(p.x, p.y) = p
}

/// The x-coordinate of a newly constructed point.
theorem point2_new_x[T](x: T, y: T) {
    Point2.new(x, y).x = x
}

/// The y-coordinate of a newly constructed point.
theorem point2_new_y[T](x: T, y: T) {
    Point2.new(x, y).y = y
}

attributes Point2[T: AddCommMonoid] {
    /// The origin of the coordinate plane.
    let zero: Point2[T] = point2_zero[T]

    /// The componentwise sum of two points.
    define add(self, other: Point2[T]) -> Point2[T] {
        Point2.new(self.x + other.x, self.y + other.y)
    }
}

/// The x-coordinate of the origin.
theorem point2_zero_x[T: AddCommMonoid] {
    point2_zero[T].x = T.0
}

/// The y-coordinate of the origin.
theorem point2_zero_y[T: AddCommMonoid] {
    point2_zero[T].y = T.0
}

/// The x-coordinate of a point sum.
theorem point2_add_x[T: AddCommMonoid](p: Point2[T], q: Point2[T]) {
    p.add(q).x = p.x + q.x
}

/// The y-coordinate of a point sum.
theorem point2_add_y[T: AddCommMonoid](p: Point2[T], q: Point2[T]) {
    p.add(q).y = p.y + q.y
}

/// Point addition is commutative.
theorem point2_add_comm[T: AddCommMonoid](p: Point2[T], q: Point2[T]) {
    p.add(q) = q.add(p)
} by {
    let lhs = p.add(q)
    let rhs = q.add(p)
    rhs.x = q.x + p.x
    rhs.y = q.y + p.y
    point2_ext(lhs, rhs)
}

/// Point addition is associative.
theorem point2_add_assoc[T: AddCommMonoid](p: Point2[T], q: Point2[T], r: Point2[T]) {
    p.add(q).add(r) = p.add(q.add(r))
} by {
    let lhs = p.add(q).add(r)
    let rhs = p.add(q.add(r))
    lhs.x = (p.x + q.x) + r.x
    rhs.x = p.x + (q.x + r.x)
    lhs.x = rhs.x
    lhs.y = (p.y + q.y) + r.y
    rhs.y = p.y + (q.y + r.y)
    lhs.y = rhs.y
    point2_ext(lhs, rhs)
}

/// Adding the origin on the right leaves a point unchanged.
theorem point2_add_zero_right[T: AddCommMonoid](p: Point2[T]) {
    p.add(point2_zero[T]) = p
} by {
    let lhs = p.add(point2_zero[T])
    point2_ext(lhs, p)
}

/// Adding the origin on the left leaves a point unchanged.
theorem point2_add_zero_left[T: AddCommMonoid](p: Point2[T]) {
    point2_zero[T].add(p) = p
} by {
    let lhs = point2_zero[T].add(p)
    point2_ext(lhs, p)
}

attributes Point2[T: AddCommGroup] {
    /// The componentwise additive inverse of a point.
    define neg(self) -> Point2[T] {
        Point2.new(-self.x, -self.y)
    }

    /// The componentwise difference of two points.
    define sub(self, other: Point2[T]) -> Point2[T] {
        Point2.new(self.x - other.x, self.y - other.y)
    }

    /// Translation by a coordinate displacement.
    define translate(self, displacement: Point2[T]) -> Point2[T] {
        self.add(displacement)
    }
}

/// The x-coordinate of a point inverse.
theorem point2_neg_x[T: AddCommGroup](p: Point2[T]) {
    p.neg.x = -p.x
}

/// The y-coordinate of a point inverse.
theorem point2_neg_y[T: AddCommGroup](p: Point2[T]) {
    p.neg.y = -p.y
}

/// The x-coordinate of a point difference.
theorem point2_sub_x[T: AddCommGroup](p: Point2[T], q: Point2[T]) {
    p.sub(q).x = p.x - q.x
}

/// The y-coordinate of a point difference.
theorem point2_sub_y[T: AddCommGroup](p: Point2[T], q: Point2[T]) {
    p.sub(q).y = p.y - q.y
}

/// The x-coordinate of a translated point.
theorem point2_translate_x[T: AddCommGroup](p: Point2[T], displacement: Point2[T]) {
    p.translate(displacement).x = p.x + displacement.x
}

/// The y-coordinate of a translated point.
theorem point2_translate_y[T: AddCommGroup](p: Point2[T], displacement: Point2[T]) {
    p.translate(displacement).y = p.y + displacement.y
}

/// A point minus itself is the origin.
theorem point2_sub_self[T: AddCommGroup](p: Point2[T]) {
    p.sub(p) = point2_zero[T]
} by {
    let lhs = p.sub(p)
    lhs.x = p.x - p.x
    lhs.x = T.0
    lhs.y = p.y - p.y
    lhs.y = T.0
    point2_ext(lhs, point2_zero[T])
}

/// Adding a point and its additive inverse gives the origin.
theorem point2_add_neg_right[T: AddCommGroup](p: Point2[T]) {
    p.add(p.neg) = point2_zero[T]
} by {
    let lhs = p.add(p.neg)
    lhs.x = p.x + -p.x
    lhs.x = T.0
    lhs.y = p.y + -p.y
    lhs.y = T.0
    point2_ext(lhs, point2_zero[T])
}

attributes Point2[T: CommRing] {
    /// Scalar multiplication of both coordinates.
    define smul(self, scalar: T) -> Point2[T] {
        Point2.new(scalar * self.x, scalar * self.y)
    }

    /// The dot product of two coordinate points.
    define dot(self, other: Point2[T]) -> T {
        self.x * other.x + self.y * other.y
    }

    /// The signed two-dimensional cross product determinant.
    define cross(self, other: Point2[T]) -> T {
        self.x * other.y - self.y * other.x
    }

    /// The squared coordinate norm.
    define norm_sq(self) -> T {
        self.dot(self)
    }

    /// The squared coordinate distance to another point.
    define dist_sq(self, other: Point2[T]) -> T {
        self.sub(other).norm_sq
    }

    /// The point with parameter `t` on the directed line to `other`.
    define param_line(self, other: Point2[T], t: T) -> Point2[T] {
        self.add(other.sub(self).smul(t))
    }

    /// The signed orientation determinant of three points.
    define orientation(self, b: Point2[T], c: Point2[T]) -> T {
        b.sub(self).cross(c.sub(self))
    }

    /// True when three points are collinear.
    define collinear(self, b: Point2[T], c: Point2[T]) -> Bool {
        self.orientation(b, c) = T.0
    }
}

/// The x-coordinate of a scalar multiple.
theorem point2_smul_x[T: CommRing](p: Point2[T], scalar: T) {
    p.smul(scalar).x = scalar * p.x
}

/// The y-coordinate of a scalar multiple.
theorem point2_smul_y[T: CommRing](p: Point2[T], scalar: T) {
    p.smul(scalar).y = scalar * p.y
}

/// The dot product is symmetric.
theorem point2_dot_comm[T: CommRing](p: Point2[T], q: Point2[T]) {
    p.dot(q) = q.dot(p)
} by {
}

/// The cross product of a point with itself is zero.
theorem point2_cross_self[T: CommRing](p: Point2[T]) {
    p.cross(p) = T.0
} by {
    p.x * p.y = p.y * p.x
}

/// The cross product with the origin on the left is zero.
theorem point2_cross_zero_left[T: CommRing](p: Point2[T]) {
    point2_zero[T].cross(p) = T.0
} by {
    mul_zero_left[T](p.y)
    mul_zero_left[T](p.x)
    T.0 - T.0 = T.0
}

/// The cross product with the origin on the right is zero.
theorem point2_cross_zero_right[T: CommRing](p: Point2[T]) {
    p.cross(point2_zero[T]) = T.0
} by {
    mul_zero_right[T](p.x)
    mul_zero_right[T](p.y)
    T.0 - T.0 = T.0
}

/// Repeating the first two points gives zero orientation.
theorem point2_orientation_same_first[T: CommRing](p: Point2[T], q: Point2[T]) {
    p.orientation(p, q) = T.0
} by {
    p.sub(p) = point2_zero[T]
    point2_cross_zero_left(q.sub(p))
}

/// Repeating the first and third points gives zero orientation.
theorem point2_orientation_same_second[T: CommRing](p: Point2[T], q: Point2[T]) {
    p.orientation(q, p) = T.0
} by {
    p.sub(p) = point2_zero[T]
    point2_cross_zero_right(q.sub(p))
}

/// Repeating the last two points gives zero orientation.
theorem point2_orientation_same_third[T: CommRing](p: Point2[T], q: Point2[T]) {
    p.orientation(q, q) = T.0
} by {
}

attributes Point2[T: OrderedField] {
    /// True when three points form a strict left turn.
    define left_turn(self, b: Point2[T], c: Point2[T]) -> Bool {
        self.orientation(b, c) > T.0
    }

    /// True when three points form a strict right turn.
    define right_turn(self, b: Point2[T], c: Point2[T]) -> Bool {
        self.orientation(b, c) < T.0
    }
}

/// Left turns are positive orientations.
theorem point2_left_turn_eq_orientation_pos[T: OrderedField](a: Point2[T], b: Point2[T], c: Point2[T]) {
    a.left_turn(b, c) = (a.orientation(b, c) > T.0)
}

/// Right turns are negative orientations.
theorem point2_right_turn_eq_orientation_neg[T: OrderedField](a: Point2[T], b: Point2[T], c: Point2[T]) {
    a.right_turn(b, c) = (a.orientation(b, c) < T.0)
}
