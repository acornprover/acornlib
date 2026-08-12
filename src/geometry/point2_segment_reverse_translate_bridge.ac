/// Bridges combining Point2 segment reversal and translation.

from ordered_field import OrderedField
from geometry.point2 import Point2
from geometry.point2_segment_intersection import Point2Segment, point2_segment_ext,
    point2_segment_reverse_end, point2_segment_reverse_start,
    point2_segment_shares_endpoint_reverse_left,
    point2_segment_shares_endpoint_reverse_right,
    point2_segment_shares_endpoint_translate, point2_segment_translate_end,
    point2_segment_translate_start, point2_segments_intersect_reverse_left,
    point2_segments_intersect_reverse_right, point2_segments_intersect_translate

/// Translating a segment and then reversing it agrees with reversing and then translating.
theorem point2_segment_translate_reverse[T: OrderedField](s: Point2Segment[T], v: Point2[T]) {
    s.translate(v).reverse = s.reverse.translate(v)
} by {
    let lhs = s.translate(v).reverse
    let rhs = s.reverse.translate(v)
    point2_segment_reverse_start(s.translate(v))
    point2_segment_translate_end(s, v)
    lhs.start = s.end.translate(v)
    point2_segment_translate_start(s.reverse, v)
    point2_segment_reverse_start(s)
    rhs.start = s.end.translate(v)
    lhs.start = rhs.start
    point2_segment_reverse_end(s.translate(v))
    point2_segment_translate_start(s, v)
    lhs.end = s.start.translate(v)
    point2_segment_translate_end(s.reverse, v)
    point2_segment_reverse_end(s)
    rhs.end = s.start.translate(v)
    lhs.end = rhs.end
    point2_segment_ext(lhs, rhs)
}

/// Reversing a segment and then translating it agrees with translating and then reversing.
theorem point2_segment_reverse_translate[T: OrderedField](s: Point2Segment[T], v: Point2[T]) {
    s.reverse.translate(v) = s.translate(v).reverse
} by {
    point2_segment_translate_reverse(s, v)
}

/// Reversing both segments preserves endpoint sharing.
theorem point2_segment_shares_endpoint_reverse_both[T: OrderedField](s: Point2Segment[T], t: Point2Segment[T]) {
    s.reverse.shares_endpoint(t.reverse) = s.shares_endpoint(t)
} by {
    point2_segment_shares_endpoint_reverse_left(s, t.reverse)
    point2_segment_shares_endpoint_reverse_right(s, t)
}

/// Reversing and translating both segments preserves endpoint sharing.
theorem point2_segment_shares_endpoint_reverse_translate[T: OrderedField](
    s: Point2Segment[T],
    t: Point2Segment[T],
    v: Point2[T]
) {
    s.reverse.translate(v).shares_endpoint(t.reverse.translate(v)) = s.shares_endpoint(t)
} by {
    point2_segment_shares_endpoint_translate(s.reverse, t.reverse, v)
    point2_segment_shares_endpoint_reverse_both(s, t)
}

/// Reversing both segments preserves intersection.
theorem point2_segments_intersect_reverse_both[T: OrderedField](s: Point2Segment[T], t: Point2Segment[T]) {
    s.reverse.intersects(t.reverse) = s.intersects(t)
} by {
    point2_segments_intersect_reverse_left(s, t.reverse)
    point2_segments_intersect_reverse_right(s, t)
}

/// Reversing and translating both segments preserves intersection.
theorem point2_segments_intersect_reverse_translate[T: OrderedField](
    s: Point2Segment[T],
    t: Point2Segment[T],
    v: Point2[T]
) {
    s.reverse.translate(v).intersects(t.reverse.translate(v)) = s.intersects(t)
} by {
    point2_segments_intersect_translate(s.reverse, t.reverse, v)
    point2_segments_intersect_reverse_both(s, t)
}
