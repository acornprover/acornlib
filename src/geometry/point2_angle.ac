/// Angle infrastructure for the coordinate plane: Cauchy–Schwarz for the
/// point dot product, the Euclidean length of a point, the cosine of the
/// angle between two vectors, its bounds, and the law of cosines in
/// angle-and-side form.
///
/// Status:
///   - The squared Cauchy–Schwarz inequality `(u·v)² ≤ |u|²|v|²` is proved
///     over any ordered field, via Lagrange's identity
///     `|u|²|v|² = (u·v)² + (u×v)²`.
///   - `point2_norm` is the real-valued Euclidean length of a point: the
///     nonnegative square root of its squared norm (the library's `sqrt`
///     is `Option`-valued, so the length is extracted as a value with a
///     zero default, in the style of `log_or_zero`).
///   - `cos_angle(u, v)` is the cosine of the angle between two nonzero
///     vectors: `u·v / (|u| |v|)`.  For nonzero vectors it is bounded by
///     `-1` and `1` (Cauchy–Schwarz).
///   - The law of cosines in angle-and-side form at vertex `a`:
///     `|b-c|² = |b-a|² + |c-a|² − 2|b-a||c-a| (∠BAC).cos` with
///     `(∠BAC).cos = cos_angle(b-a, c-a)`.

from comm_ring import CommRing
from algebra.add_comm_group import AddCommGroup, sub_eq_zero_imp_eq
from data.basic.witness import choose_or_default, choose_or_default_spec
from data.basic.logic import exists_intro
from real import Real, sqrt_mul_self, sqrt_value_nonneg, square_le_square_of_nonneg, abs_of_nonneg,
    abs_gte_zero, lte_abs, abs_neg, square_nonneg, mul_nonneg,
    mul_abs, div_le_of_mul_le, mul_div_cancel, mul_left_cancel, mul_inverse,
    mul_le_mul_pos_right, lte_add_right, lte_add_left, add_lte_add, only_abs_zero_eq_zero,
    prod_eq_to_div_eq
from order import lte_trans, lt_trans, lte_antisymm, not_gt_imp_lte,
    lt_imp_ne, lt_of_lt_of_lte, lt_imp_lte
from algebra.add_ordered_group import add_inequality
from algebra.field.field import mul_not_zero
from ordered_field import OrderedField, squares_are_nonnegative, mul_pos_pos,
    inverse_of_positive_is_positive
from geometry.point2 import Point2, point2_zero, point2_dot_comm, point2_ext,
    point2_zero_x, point2_zero_y, point2_sub_x, point2_sub_y
from geometry.analytic_geometry import real_sqrt_unique
from geometry.point2_algebra import point2_norm_sq_nonneg, point2_dist_sq_comm,
    point2_norm_sq_sub_expansion, point2_norm_sq_neg, point2_cross_swap,
    point2_cross_neg_right, point2_cross_sub_right, point2_cross_self,
    point2_sub_reverse_neg
from geometry.point2_metric import point2_dist_sq_nonneg,
    point2_dist_sq_eq_norm_sq_sub_reverse
from geometry.point2_law_of_cosines import point2_law_of_cosines_dist_sq,
    point2_double_dot_from_dist_sq
from geometry.point2_heron import point2_sub_same_base,
    point2_sub_eq_sub_imp_sub_eq, point2_sub_sub_eq_sub_add

numerals Real

// ---------------------------------------------------------------------------
// Cauchy–Schwarz for the point dot product.
// ---------------------------------------------------------------------------

/// Lagrange's identity for two coordinate vectors: the product of the squared
/// norms is the sum of the squared dot product and the squared cross product.
theorem point2_norm_sq_mul_eq_dot_sq_add_cross_sq[T: CommRing](u: Point2[T], v: Point2[T]) {
    u.norm_sq * v.norm_sq = u.dot(v) * u.dot(v) + u.cross(v) * u.cross(v)
} by {
    u.norm_sq = u.x * u.x + u.y * u.y
    v.norm_sq = v.x * v.x + v.y * v.y
    u.dot(v) = u.x * v.x + u.y * v.y
    u.cross(v) = u.x * v.y - u.y * v.x
    (u.x * u.x + u.y * u.y) * (v.x * v.x + v.y * v.y) =
        (u.x * v.x + u.y * v.y) * (u.x * v.x + u.y * v.y) +
        (u.x * v.y - u.y * v.x) * (u.x * v.y - u.y * v.x)
    u.norm_sq * v.norm_sq =
        (u.x * v.x + u.y * v.y) * (u.x * v.x + u.y * v.y) +
        (u.x * v.y - u.y * v.x) * (u.x * v.y - u.y * v.x)
    u.norm_sq * v.norm_sq = u.dot(v) * u.dot(v) + u.cross(v) * u.cross(v)
}

/// Adding a nonnegative element to the right does not decrease a sum.
theorem point2_add_nonneg_right[T: OrderedField](x: T, y: T) {
    y >= T.0 implies x <= x + y
} by {
    if y >= T.0 {
        add_inequality(x, x, T.0, y)
        x + T.0 <= x + y
        x + T.0 = x
        x <= x + y
    }
}

/// Cauchy–Schwarz in squared form for the point dot product: the square of
/// the dot product is at most the product of the squared norms.
theorem point2_cauchy_schwarz_sq[T: OrderedField](u: Point2[T], v: Point2[T]) {
    u.dot(v) * u.dot(v) <= u.norm_sq * v.norm_sq
} by {
    point2_norm_sq_mul_eq_dot_sq_add_cross_sq(u, v)
    u.norm_sq * v.norm_sq = u.dot(v) * u.dot(v) + u.cross(v) * u.cross(v)
    squares_are_nonnegative[T](u.cross(v))
    u.cross(v) * u.cross(v) >= T.0
    point2_add_nonneg_right(u.dot(v) * u.dot(v), u.cross(v) * u.cross(v))
    u.dot(v) * u.dot(v) <= u.dot(v) * u.dot(v) + u.cross(v) * u.cross(v)
    u.dot(v) * u.dot(v) <= u.norm_sq * v.norm_sq
}

// ---------------------------------------------------------------------------
// The Euclidean length of a point.
// ---------------------------------------------------------------------------

/// The Euclidean length of a point: the nonnegative square root of its
/// squared norm (zero outside the nonnegative domain).
define point2_norm(p: Point2[Real]) -> Real {
    match (p.norm_sq).sqrt {
        Option.some(y) {
            y
        }
        Option.none {
            Real.0
        }
    }
}

/// The length of a point is nonnegative and its square is the squared norm.
theorem point2_norm_spec(p: Point2[Real]) {
    point2_norm(p) >= Real.0 and point2_norm(p) * point2_norm(p) = p.norm_sq
} by {
    point2_norm_sq_nonneg(p)
    p.norm_sq >= Real.0
    sqrt_mul_self(p.norm_sq)
    exists(y: Real) {
        (p.norm_sq).sqrt = Option.some(y) and y * y = p.norm_sq
    }
    let y: Real satisfy {
        (p.norm_sq).sqrt = Option.some(y) and y * y = p.norm_sq
    }
    sqrt_value_nonneg(p.norm_sq, y)
    y >= Real.0
    match (p.norm_sq).sqrt {
        Option.some(v) {
            (p.norm_sq).sqrt = Option.some(v)
            (p.norm_sq).sqrt = Option.some(y)
            v = y
            point2_norm(p) = y
            point2_norm(p) >= Real.0
            point2_norm(p) * point2_norm(p) = y * y
            point2_norm(p) * point2_norm(p) = p.norm_sq
            point2_norm(p) >= Real.0 and point2_norm(p) * point2_norm(p) = p.norm_sq
        }
        Option.none {
            (p.norm_sq).sqrt = Option.none[Real]
            (p.norm_sq).sqrt = Option.some(y)
            false
        }
    }
}

/// The length of a point is nonnegative.
theorem point2_norm_nonneg(p: Point2[Real]) {
    point2_norm(p) >= Real.0
} by {
    point2_norm_spec(p)
}

/// The square of the length of a point is its squared norm.
theorem point2_norm_sq(p: Point2[Real]) {
    point2_norm(p) * point2_norm(p) = p.norm_sq
} by {
    point2_norm_spec(p)
}

/// The square of a nonzero real is positive.
theorem real_square_pos(x: Real) {
    x != Real.0 implies x * x > Real.0
} by {
    if x != Real.0 {
        square_nonneg(x)
        x * x >= Real.0
        mul_not_zero[Real](x, x)
        x * x != Real.0
        if x * x > Real.0 {
        } else {
            not_gt_imp_lte(x * x, Real.0)
            x * x <= Real.0
            lte_antisymm(Real.0, x * x)
            Real.0 = x * x
            x * x = Real.0
            false
        }
    }
}

/// Adding a positive real to a nonnegative one is positive.
theorem real_add_nonneg_pos(a: Real, b: Real) {
    a >= Real.0 and b > Real.0 implies a + b > Real.0
} by {
    if a >= Real.0 and b > Real.0 {
        lte_add_right(Real.0, a, b)
        Real.0 + b <= a + b
        Real.0 + b = b
        b <= a + b
        Real.0 < b
        lt_of_lt_of_lte(Real.0, b, a + b)
        Real.0 < a + b
        a + b > Real.0
    }
}

/// A nonnegative real that is not zero is positive.
theorem real_nonneg_ne_zero_imp_pos(x: Real) {
    x >= Real.0 and x != Real.0 implies x > Real.0
} by {
    if x >= Real.0 and x != Real.0 {
        if x > Real.0 {
        } else {
            not_gt_imp_lte(x, Real.0)
            x <= Real.0
            lte_antisymm(Real.0, x)
            Real.0 = x
            x = Real.0
            false
        }
    }
}

/// The squared norm of a nonzero point is positive.
theorem point2_norm_sq_pos(p: Point2[Real]) {
    p != point2_zero[Real] implies p.norm_sq > Real.0
} by {
    if p != point2_zero[Real] {
        if p.x = Real.0 {
            if p.y = Real.0 {
                point2_zero_x[Real]
                point2_zero[Real].x = Real.0
                p.x = point2_zero[Real].x
                point2_zero_y[Real]
                point2_zero[Real].y = Real.0
                p.y = point2_zero[Real].y
                point2_ext(p, point2_zero[Real])
                p = point2_zero[Real]
                false
            }
            p.y != Real.0
            real_square_pos(p.y)
            p.y * p.y > Real.0
            p.x * p.x = Real.0 * Real.0
            Real.0 * Real.0 = Real.0
            p.x * p.x = Real.0
            p.x * p.x + p.y * p.y = Real.0 + p.y * p.y
            Real.0 + p.y * p.y = p.y * p.y
            p.x * p.x + p.y * p.y = p.y * p.y
            p.norm_sq = p.x * p.x + p.y * p.y
            p.norm_sq = p.y * p.y
            p.norm_sq > Real.0
        } else {
            p.x != Real.0
            real_square_pos(p.x)
            p.x * p.x > Real.0
            square_nonneg(p.y)
            p.y * p.y >= Real.0
            real_add_nonneg_pos(p.y * p.y, p.x * p.x)
            p.y * p.y + p.x * p.x > Real.0
            p.norm_sq = p.x * p.x + p.y * p.y
            p.norm_sq = p.y * p.y + p.x * p.x
            p.norm_sq > Real.0
        }
    }
}

/// The length of a nonzero point is positive.
theorem point2_norm_pos(p: Point2[Real]) {
    p != point2_zero[Real] implies point2_norm(p) > Real.0
} by {
    if p != point2_zero[Real] {
        point2_norm_sq_pos(p)
        p.norm_sq > Real.0
        point2_norm_sq(p)
        point2_norm(p) * point2_norm(p) = p.norm_sq
        point2_norm(p) * point2_norm(p) > Real.0
        point2_norm_nonneg(p)
        point2_norm(p) >= Real.0
        if point2_norm(p) = Real.0 {
            point2_norm(p) * point2_norm(p) = Real.0 * Real.0
            Real.0 * Real.0 = Real.0
            point2_norm(p) * point2_norm(p) = Real.0
            false
        }
        point2_norm(p) != Real.0
        real_nonneg_ne_zero_imp_pos(point2_norm(p))
        point2_norm(p) > Real.0
    }
}

// ---------------------------------------------------------------------------
// The cosine of the angle between two vectors.
// ---------------------------------------------------------------------------

/// The cosine of the angle between two vectors: the dot product over the
/// product of the lengths.  The result is total; the interesting bounds hold
/// for nonzero vectors.
define cos_angle(u: Point2[Real], v: Point2[Real]) -> Real {
    u.dot(v) / (point2_norm(u) * point2_norm(v))
}

/// The absolute value of the dot product is at most the product of the
/// lengths of the two vectors.
theorem point2_cauchy_schwarz_abs(u: Point2[Real], v: Point2[Real]) {
    (u.dot(v)).abs <= point2_norm(u) * point2_norm(v)
} by {
    point2_cauchy_schwarz_sq[Real](u, v)
    u.dot(v) * u.dot(v) <= u.norm_sq * v.norm_sq
    point2_norm_sq(u)
    point2_norm(u) * point2_norm(u) = u.norm_sq
    point2_norm_sq(v)
    point2_norm(v) * point2_norm(v) = v.norm_sq
    (point2_norm(u) * point2_norm(v)) * (point2_norm(u) * point2_norm(v)) =
        (point2_norm(u) * point2_norm(u)) * (point2_norm(v) * point2_norm(v))
    (point2_norm(u) * point2_norm(u)) * (point2_norm(v) * point2_norm(v)) =
        u.norm_sq * v.norm_sq
    (point2_norm(u) * point2_norm(v)) * (point2_norm(u) * point2_norm(v)) =
        u.norm_sq * v.norm_sq
    u.dot(v) * u.dot(v) <= (point2_norm(u) * point2_norm(v)) * (point2_norm(u) * point2_norm(v))
    mul_abs(u.dot(v), u.dot(v))
    (u.dot(v)).abs * (u.dot(v)).abs = ((u.dot(v)) * (u.dot(v))).abs
    square_nonneg(u.dot(v))
    u.dot(v) * u.dot(v) >= Real.0
    abs_of_nonneg(u.dot(v) * u.dot(v))
    ((u.dot(v)) * (u.dot(v))).abs = u.dot(v) * u.dot(v)
    (u.dot(v)).abs * (u.dot(v)).abs = u.dot(v) * u.dot(v)
    point2_norm_nonneg(u)
    point2_norm(u) >= Real.0
    point2_norm_nonneg(v)
    point2_norm(v) >= Real.0
    mul_nonneg(point2_norm(u), point2_norm(v))
    point2_norm(u) * point2_norm(v) >= Real.0
    abs_gte_zero(u.dot(v))
    (u.dot(v)).abs >= Real.0
    square_le_square_of_nonneg((u.dot(v)).abs, point2_norm(u) * point2_norm(v))
    (u.dot(v)).abs <= point2_norm(u) * point2_norm(v)
}

/// Cauchy–Schwarz for the point dot product: the dot product is at most the
/// product of the lengths of the two vectors.
theorem point2_cauchy_schwarz(u: Point2[Real], v: Point2[Real]) {
    u.dot(v) <= point2_norm(u) * point2_norm(v)
} by {
    point2_cauchy_schwarz_abs(u, v)
    (u.dot(v)).abs <= point2_norm(u) * point2_norm(v)
    lte_abs(u.dot(v))
    u.dot(v) <= (u.dot(v)).abs
    lte_trans(u.dot(v), (u.dot(v)).abs, point2_norm(u) * point2_norm(v))
}

/// An absolute value bounded by a real is at most that real.
theorem real_abs_le_imp_lte(x: Real, y: Real) {
    x.abs <= y implies x <= y
} by {
    lte_abs(x)
    x <= x.abs
    lte_trans(x, x.abs, y)
}

/// An absolute value bounded by a real bounds the number from below.
theorem real_abs_le_imp_neg_lte(x: Real, y: Real) {
    x.abs <= y implies -y <= x
} by {
    lte_abs(-x)
    -x <= (-x).abs
    abs_neg(x)
    (-x).abs = x.abs
    lte_trans(-x, x.abs, y)
    -x <= y
    lte_add_right(-x, y, x)
    -x + x <= y + x
    -x + x = Real.0
    Real.0 <= y + x
    lte_add_left(Real.0, y + x, -y)
    -y + Real.0 <= -y + (y + x)
    -y + Real.0 = -y
    -y <= -y + (y + x)
    -y + y = Real.0
    -y + (y + x) = (-y + y) + x
    (-y + y) + x = Real.0 + x
    Real.0 + x = x
    -y + (y + x) = x
    -y <= x
}

/// An absolute value bounded by a real is bounded between the negation of the
/// bound and the bound.
theorem real_abs_le_imp_le_bounds(x: Real, y: Real) {
    x.abs <= y implies -y <= x and x <= y
} by {
    real_abs_le_imp_neg_lte(x, y)
    -y <= x
    real_abs_le_imp_lte(x, y)
    x <= y
}

/// The cosine of the angle between two nonzero vectors is at most one.
theorem cos_angle_le_one(u: Point2[Real], v: Point2[Real]) {
    u != point2_zero[Real] and v != point2_zero[Real] implies cos_angle(u, v) <= Real.1
} by {
    if u != point2_zero[Real] and v != point2_zero[Real] {
        point2_cauchy_schwarz(u, v)
        u.dot(v) <= point2_norm(u) * point2_norm(v)
        point2_norm_pos(u)
        Real.0 < point2_norm(u)
        point2_norm_pos(v)
        Real.0 < point2_norm(v)
        mul_pos_pos[Real](point2_norm(u), point2_norm(v))
        Real.0 < point2_norm(u) * point2_norm(v)
        point2_norm(u) * point2_norm(v) > Real.0
        inverse_of_positive_is_positive[Real](point2_norm(u) * point2_norm(v))
        Real.0 < (point2_norm(u) * point2_norm(v)).inverse
        (point2_norm(u) * point2_norm(v)).inverse > Real.0
        mul_le_mul_pos_right(u.dot(v), point2_norm(u) * point2_norm(v),
            (point2_norm(u) * point2_norm(v)).inverse)
        u.dot(v) * (point2_norm(u) * point2_norm(v)).inverse <= (point2_norm(u) * point2_norm(v)) * (point2_norm(u) * point2_norm(v)).inverse
        lt_imp_ne[Real](Real.0, point2_norm(u) * point2_norm(v))
        point2_norm(u) * point2_norm(v) != Real.0
        mul_inverse(point2_norm(u) * point2_norm(v))
        (point2_norm(u) * point2_norm(v)) * (point2_norm(u) * point2_norm(v)).inverse = Real.1
        u.dot(v) * (point2_norm(u) * point2_norm(v)).inverse <= Real.1
        u.dot(v) / (point2_norm(u) * point2_norm(v)) <= Real.1
        cos_angle(u, v) = u.dot(v) / (point2_norm(u) * point2_norm(v))
        cos_angle(u, v) <= Real.1
    }
}

/// The cosine of the angle between two nonzero vectors is at least negative
/// one.
theorem cos_angle_ge_neg_one(u: Point2[Real], v: Point2[Real]) {
    u != point2_zero[Real] and v != point2_zero[Real] implies -Real.1 <= cos_angle(u, v)
} by {
    if u != point2_zero[Real] and v != point2_zero[Real] {
        point2_cauchy_schwarz_abs(u, v)
        (u.dot(v)).abs <= point2_norm(u) * point2_norm(v)
        real_abs_le_imp_neg_lte(u.dot(v), point2_norm(u) * point2_norm(v))
        -(point2_norm(u) * point2_norm(v)) <= u.dot(v)
        -Real.1 * (point2_norm(u) * point2_norm(v)) = -(point2_norm(u) * point2_norm(v))
        -Real.1 * (point2_norm(u) * point2_norm(v)) <= u.dot(v)
        point2_norm_pos(u)
        Real.0 < point2_norm(u)
        point2_norm_pos(v)
        Real.0 < point2_norm(v)
        mul_pos_pos[Real](point2_norm(u), point2_norm(v))
        Real.0 < point2_norm(u) * point2_norm(v)
        point2_norm(u) * point2_norm(v) > Real.0
        div_le_of_mul_le(-Real.1, point2_norm(u) * point2_norm(v), u.dot(v))
        -Real.1 <= u.dot(v) / (point2_norm(u) * point2_norm(v))
        cos_angle(u, v) = u.dot(v) / (point2_norm(u) * point2_norm(v))
        -Real.1 <= cos_angle(u, v)
    }
}

/// The cosine of the angle between two nonzero vectors lies between negative
/// one and one.
theorem cos_angle_bounds(u: Point2[Real], v: Point2[Real]) {
    u != point2_zero[Real] and v != point2_zero[Real] implies
    -Real.1 <= cos_angle(u, v) and cos_angle(u, v) <= Real.1
} by {
    cos_angle_ge_neg_one(u, v)
    -Real.1 <= cos_angle(u, v)
    cos_angle_le_one(u, v)
    cos_angle(u, v) <= Real.1
}

// ---------------------------------------------------------------------------
// The law of cosines in angle-and-side form.
// ---------------------------------------------------------------------------

/// The product of the lengths times the cosine of the angle is the dot
/// product, for nonzero vectors.
theorem point2_dot_eq_norm_mul_cos_angle(u: Point2[Real], v: Point2[Real]) {
    u != point2_zero[Real] and v != point2_zero[Real] implies
    point2_norm(u) * point2_norm(v) * cos_angle(u, v) = u.dot(v)
} by {
    if u != point2_zero[Real] and v != point2_zero[Real] {
        point2_norm_pos(u)
        Real.0 < point2_norm(u)
        point2_norm_pos(v)
        Real.0 < point2_norm(v)
        mul_pos_pos[Real](point2_norm(u), point2_norm(v))
        Real.0 < point2_norm(u) * point2_norm(v)
        point2_norm(u) * point2_norm(v) > Real.0
        lt_imp_ne[Real](Real.0, point2_norm(u) * point2_norm(v))
        point2_norm(u) * point2_norm(v) != Real.0
        mul_div_cancel(u.dot(v), point2_norm(u) * point2_norm(v))
        point2_norm(u) * point2_norm(v) * (u.dot(v) / (point2_norm(u) * point2_norm(v))) = u.dot(v)
        cos_angle(u, v) = u.dot(v) / (point2_norm(u) * point2_norm(v))
        point2_norm(u) * point2_norm(v) * cos_angle(u, v) = u.dot(v)
    }
}

/// A point difference equal to the origin means the points are equal.
theorem point2_sub_eq_zero_imp_eq[T: AddCommGroup](p: Point2[T], q: Point2[T]) {
    p.sub(q) = point2_zero[T] implies p = q
} by {
    if p.sub(q) = point2_zero[T] {
        point2_sub_x(p, q)
        p.sub(q).x = p.x - q.x
        point2_zero_x[T]
        point2_zero[T].x = T.0
        p.x - q.x = T.0
        sub_eq_zero_imp_eq[T](p.x, q.x)
        p.x = q.x
        point2_sub_y(p, q)
        p.sub(q).y = p.y - q.y
        point2_zero_y[T]
        point2_zero[T].y = T.0
        p.y - q.y = T.0
        sub_eq_zero_imp_eq[T](p.y, q.y)
        p.y = q.y
        point2_ext(p, q)
        p = q
    }
}

/// Twice a real is two times the real.
theorem point2_double_eq_two_mul(x: Real) {
    x + x = (Real.1 + Real.1) * x
} by {
    (Real.1 + Real.1) * x = Real.1 * x + Real.1 * x
    Real.1 * x = x
    (Real.1 + Real.1) * x = x + x
    x + x = (Real.1 + Real.1) * x
}

/// The law of cosines in angle-and-side form at vertex `a`: for a nondegenerate
/// triangle with `u = b - a` and `v = c - a`,
/// `|b-c|² = |b-a|² + |c-a|² − 2|b-a||c-a| (∠BAC).cos`.
theorem point2_law_of_cosines_angle(a: Point2[Real], b: Point2[Real], c: Point2[Real]) {
    b != a and c != a implies
    b.dist_sq(c) =
        a.dist_sq(b) + a.dist_sq(c) -
        (Real.1 + Real.1) * (point2_norm(b.sub(a)) * point2_norm(c.sub(a)) *
            cos_angle(b.sub(a), c.sub(a)))
} by {
    if b != a and c != a {
        if b.sub(a) = point2_zero[Real] {
            point2_sub_eq_zero_imp_eq[Real](b, a)
            b = a
            false
        }
        b.sub(a) != point2_zero[Real]
        if c.sub(a) = point2_zero[Real] {
            point2_sub_eq_zero_imp_eq[Real](c, a)
            c = a
            false
        }
        c.sub(a) != point2_zero[Real]
        point2_dot_eq_norm_mul_cos_angle(b.sub(a), c.sub(a))
        point2_norm(b.sub(a)) * point2_norm(c.sub(a)) * cos_angle(b.sub(a), c.sub(a)) =
            b.sub(a).dot(c.sub(a))
        point2_law_of_cosines_dist_sq(a, b, c)
        b.dist_sq(c) =
            a.dist_sq(b) + a.dist_sq(c) -
            b.sub(a).dot(c.sub(a)) - b.sub(a).dot(c.sub(a))
        b.dist_sq(c) =
            a.dist_sq(b) + a.dist_sq(c) -
            point2_norm(b.sub(a)) * point2_norm(c.sub(a)) * cos_angle(b.sub(a), c.sub(a)) -
            point2_norm(b.sub(a)) * point2_norm(c.sub(a)) * cos_angle(b.sub(a), c.sub(a))
        point2_sub_sub_eq_sub_add(
            a.dist_sq(b) + a.dist_sq(c),
            point2_norm(b.sub(a)) * point2_norm(c.sub(a)) * cos_angle(b.sub(a), c.sub(a)),
            point2_norm(b.sub(a)) * point2_norm(c.sub(a)) * cos_angle(b.sub(a), c.sub(a)))
        a.dist_sq(b) + a.dist_sq(c) -
            point2_norm(b.sub(a)) * point2_norm(c.sub(a)) * cos_angle(b.sub(a), c.sub(a)) -
            point2_norm(b.sub(a)) * point2_norm(c.sub(a)) * cos_angle(b.sub(a), c.sub(a)) =
            a.dist_sq(b) + a.dist_sq(c) -
            (point2_norm(b.sub(a)) * point2_norm(c.sub(a)) * cos_angle(b.sub(a), c.sub(a)) +
                point2_norm(b.sub(a)) * point2_norm(c.sub(a)) * cos_angle(b.sub(a), c.sub(a)))
        point2_double_eq_two_mul(point2_norm(b.sub(a)) * point2_norm(c.sub(a)) *
            cos_angle(b.sub(a), c.sub(a)))
        point2_norm(b.sub(a)) * point2_norm(c.sub(a)) * cos_angle(b.sub(a), c.sub(a)) +
            point2_norm(b.sub(a)) * point2_norm(c.sub(a)) * cos_angle(b.sub(a), c.sub(a)) =
            (Real.1 + Real.1) * (point2_norm(b.sub(a)) * point2_norm(c.sub(a)) *
                cos_angle(b.sub(a), c.sub(a)))
        b.dist_sq(c) =
            a.dist_sq(b) + a.dist_sq(c) -
            (Real.1 + Real.1) * (point2_norm(b.sub(a)) * point2_norm(c.sub(a)) *
                cos_angle(b.sub(a), c.sub(a)))
    }
}

// ---------------------------------------------------------------------------
// The sine of the angle between two vectors and the law of sines.
// ---------------------------------------------------------------------------

/// The sine of the angle between two vectors: the absolute value of the cross
/// product over the product of the lengths.
define sin_angle(u: Point2[Real], v: Point2[Real]) -> Real {
    (u.cross(v)).abs / (point2_norm(u) * point2_norm(v))
}

/// The negation of a point has the same length.
theorem point2_norm_neg(p: Point2[Real]) {
    point2_norm(p.neg) = point2_norm(p)
} by {
    point2_norm_sq_neg(p)
    p.neg.norm_sq = p.norm_sq
    point2_norm_spec(p)
    point2_norm(p) >= Real.0 and point2_norm(p) * point2_norm(p) = p.norm_sq
    point2_norm(p) * point2_norm(p) = p.norm_sq
    point2_norm_spec(p.neg)
    point2_norm(p.neg) >= Real.0 and point2_norm(p.neg) * point2_norm(p.neg) = p.neg.norm_sq
    point2_norm(p.neg) * point2_norm(p.neg) = p.neg.norm_sq
    point2_norm(p.neg) * point2_norm(p.neg) = p.norm_sq
    point2_norm(p.neg) * point2_norm(p.neg) = point2_norm(p) * point2_norm(p)
    square_le_square_of_nonneg(point2_norm(p.neg), point2_norm(p))
    point2_norm(p.neg) <= point2_norm(p)
    square_le_square_of_nonneg(point2_norm(p), point2_norm(p.neg))
    point2_norm(p) <= point2_norm(p.neg)
    lte_antisymm(point2_norm(p.neg), point2_norm(p))
    point2_norm(p.neg) = point2_norm(p)
}

/// Cauchy–Schwarz for the cross product: the absolute value of the cross
/// product is at most the product of the lengths of the two vectors.
theorem point2_cauchy_schwarz_cross_abs(u: Point2[Real], v: Point2[Real]) {
    (u.cross(v)).abs <= point2_norm(u) * point2_norm(v)
} by {
    point2_norm_sq_mul_eq_dot_sq_add_cross_sq(u, v)
    u.norm_sq * v.norm_sq = u.dot(v) * u.dot(v) + u.cross(v) * u.cross(v)
    squares_are_nonnegative[Real](u.dot(v))
    u.dot(v) * u.dot(v) >= Real.0
    point2_add_nonneg_right(u.cross(v) * u.cross(v), u.dot(v) * u.dot(v))
    u.cross(v) * u.cross(v) <= u.cross(v) * u.cross(v) + u.dot(v) * u.dot(v)
    u.cross(v) * u.cross(v) <= u.norm_sq * v.norm_sq
    point2_norm_sq(u)
    point2_norm(u) * point2_norm(u) = u.norm_sq
    point2_norm_sq(v)
    point2_norm(v) * point2_norm(v) = v.norm_sq
    (point2_norm(u) * point2_norm(v)) * (point2_norm(u) * point2_norm(v)) =
        (point2_norm(u) * point2_norm(u)) * (point2_norm(v) * point2_norm(v))
    (point2_norm(u) * point2_norm(u)) * (point2_norm(v) * point2_norm(v)) =
        u.norm_sq * v.norm_sq
    (point2_norm(u) * point2_norm(v)) * (point2_norm(u) * point2_norm(v)) =
        u.norm_sq * v.norm_sq
    u.cross(v) * u.cross(v) <= (point2_norm(u) * point2_norm(v)) * (point2_norm(u) * point2_norm(v))
    mul_abs(u.cross(v), u.cross(v))
    (u.cross(v)).abs * (u.cross(v)).abs = ((u.cross(v)) * (u.cross(v))).abs
    square_nonneg(u.cross(v))
    u.cross(v) * u.cross(v) >= Real.0
    abs_of_nonneg(u.cross(v) * u.cross(v))
    ((u.cross(v)) * (u.cross(v))).abs = u.cross(v) * u.cross(v)
    (u.cross(v)).abs * (u.cross(v)).abs = u.cross(v) * u.cross(v)
    point2_norm_nonneg(u)
    point2_norm(u) >= Real.0
    point2_norm_nonneg(v)
    point2_norm(v) >= Real.0
    mul_nonneg(point2_norm(u), point2_norm(v))
    point2_norm(u) * point2_norm(v) >= Real.0
    abs_gte_zero(u.cross(v))
    (u.cross(v)).abs >= Real.0
    square_le_square_of_nonneg((u.cross(v)).abs, point2_norm(u) * point2_norm(v))
    (u.cross(v)).abs <= point2_norm(u) * point2_norm(v)
}

/// The product of the lengths times the sine of the angle is the absolute
/// value of the cross product, for nonzero vectors.
theorem point2_cross_eq_norm_mul_sin_angle(u: Point2[Real], v: Point2[Real]) {
    u != point2_zero[Real] and v != point2_zero[Real] implies
    point2_norm(u) * point2_norm(v) * sin_angle(u, v) = (u.cross(v)).abs
} by {
    if u != point2_zero[Real] and v != point2_zero[Real] {
        point2_norm_pos(u)
        Real.0 < point2_norm(u)
        point2_norm_pos(v)
        Real.0 < point2_norm(v)
        mul_pos_pos[Real](point2_norm(u), point2_norm(v))
        Real.0 < point2_norm(u) * point2_norm(v)
        point2_norm(u) * point2_norm(v) > Real.0
        lt_imp_ne[Real](Real.0, point2_norm(u) * point2_norm(v))
        point2_norm(u) * point2_norm(v) != Real.0
        mul_div_cancel((u.cross(v)).abs, point2_norm(u) * point2_norm(v))
        point2_norm(u) * point2_norm(v) *
            ((u.cross(v)).abs / (point2_norm(u) * point2_norm(v))) = (u.cross(v)).abs
        sin_angle(u, v) = (u.cross(v)).abs / (point2_norm(u) * point2_norm(v))
        point2_norm(u) * point2_norm(v) * sin_angle(u, v) = (u.cross(v)).abs
    }
}

/// The sine of the angle between two nonzero vectors is nonnegative.
theorem sin_angle_nonneg(u: Point2[Real], v: Point2[Real]) {
    u != point2_zero[Real] and v != point2_zero[Real] implies Real.0 <= sin_angle(u, v)
} by {
    if u != point2_zero[Real] and v != point2_zero[Real] {
        point2_norm_pos(u)
        Real.0 < point2_norm(u)
        point2_norm_pos(v)
        Real.0 < point2_norm(v)
        mul_pos_pos[Real](point2_norm(u), point2_norm(v))
        Real.0 < point2_norm(u) * point2_norm(v)
        inverse_of_positive_is_positive[Real](point2_norm(u) * point2_norm(v))
        Real.0 < (point2_norm(u) * point2_norm(v)).inverse
        lt_imp_lte[Real](Real.0, (point2_norm(u) * point2_norm(v)).inverse)
        Real.0 <= (point2_norm(u) * point2_norm(v)).inverse
        abs_gte_zero(u.cross(v))
        Real.0 <= (u.cross(v)).abs
        mul_nonneg((u.cross(v)).abs, (point2_norm(u) * point2_norm(v)).inverse)
        (u.cross(v)).abs * (point2_norm(u) * point2_norm(v)).inverse >= Real.0
        (u.cross(v)).abs / (point2_norm(u) * point2_norm(v)) >= Real.0
        sin_angle(u, v) = (u.cross(v)).abs / (point2_norm(u) * point2_norm(v))
        Real.0 <= sin_angle(u, v)
    }
}

/// The sine of the angle between two nonzero vectors is at most one.
theorem sin_angle_le_one(u: Point2[Real], v: Point2[Real]) {
    u != point2_zero[Real] and v != point2_zero[Real] implies sin_angle(u, v) <= Real.1
} by {
    if u != point2_zero[Real] and v != point2_zero[Real] {
        point2_cauchy_schwarz_cross_abs(u, v)
        (u.cross(v)).abs <= point2_norm(u) * point2_norm(v)
        point2_norm_pos(u)
        Real.0 < point2_norm(u)
        point2_norm_pos(v)
        Real.0 < point2_norm(v)
        mul_pos_pos[Real](point2_norm(u), point2_norm(v))
        Real.0 < point2_norm(u) * point2_norm(v)
        inverse_of_positive_is_positive[Real](point2_norm(u) * point2_norm(v))
        Real.0 < (point2_norm(u) * point2_norm(v)).inverse
        mul_le_mul_pos_right((u.cross(v)).abs, point2_norm(u) * point2_norm(v),
            (point2_norm(u) * point2_norm(v)).inverse)
        (u.cross(v)).abs * (point2_norm(u) * point2_norm(v)).inverse <= (point2_norm(u) * point2_norm(v)) * (point2_norm(u) * point2_norm(v)).inverse
        lt_imp_ne[Real](Real.0, point2_norm(u) * point2_norm(v))
        point2_norm(u) * point2_norm(v) != Real.0
        mul_inverse(point2_norm(u) * point2_norm(v))
        (point2_norm(u) * point2_norm(v)) * (point2_norm(u) * point2_norm(v)).inverse = Real.1
        (u.cross(v)).abs * (point2_norm(u) * point2_norm(v)).inverse <= Real.1
        (u.cross(v)).abs / (point2_norm(u) * point2_norm(v)) <= Real.1
        sin_angle(u, v) = (u.cross(v)).abs / (point2_norm(u) * point2_norm(v))
        sin_angle(u, v) <= Real.1
    }
}

/// Rotating the vertex of the side vectors preserves their cross product:
/// `(c-b)×(a-b) = (b-a)×(c-a)`.
theorem point2_cross_vertex_rotate(a: Point2[Real], b: Point2[Real], c: Point2[Real]) {
    c.sub(b).cross(a.sub(b)) = b.sub(a).cross(c.sub(a))
} by {
    let u = b.sub(a)
    let v = c.sub(a)
    point2_sub_same_base(a, b, c)
    v.sub(u) = c.sub(b)
    c.sub(b) = v.sub(u)
    point2_sub_reverse_neg(a, b)
    a.sub(b) = u.neg
    point2_cross_sub_right(v, u, u.neg)
    v.sub(u).cross(u.neg) = v.cross(u.neg) - u.cross(u.neg)
    point2_cross_neg_right(v, u)
    v.cross(u.neg) = -v.cross(u)
    point2_cross_neg_right(u, u)
    u.cross(u.neg) = -u.cross(u)
    point2_cross_self(u)
    u.cross(u) = Real.0
    -u.cross(u) = Real.0
    u.cross(u.neg) = Real.0
    v.cross(u.neg) - u.cross(u.neg) = -v.cross(u) - Real.0
    -v.cross(u) - Real.0 = -v.cross(u)
    v.sub(u).cross(u.neg) = -v.cross(u)
    point2_cross_swap(u, v)
    u.cross(v) = -v.cross(u)
    -v.cross(u) = u.cross(v)
    v.sub(u).cross(u.neg) = u.cross(v)
    c.sub(b).cross(a.sub(b)) = u.cross(v)
    c.sub(b).cross(a.sub(b)) = b.sub(a).cross(c.sub(a))
}

/// The doubled area of a triangle is the absolute value of the cross product
/// of the two side vectors meeting at any vertex.
theorem point2_area2_cross_vertex_invariant(a: Point2[Real], b: Point2[Real], c: Point2[Real]) {
    (c.sub(b).cross(a.sub(b))).abs = (b.sub(a).cross(c.sub(a))).abs
} by {
    point2_cross_vertex_rotate(a, b, c)
    c.sub(b).cross(a.sub(b)) = b.sub(a).cross(c.sub(a))
    (c.sub(b).cross(a.sub(b))).abs = (b.sub(a).cross(c.sub(a))).abs
}

/// The law of sines in sine form: for a nondegenerate triangle, a side times
/// the sine of the opposite angle is the same for each side.  With
/// `a = |b-c|` the side opposite `A = ∠BAC` and `b = |c-a|` the side opposite
/// `B = ∠CBA`, the identity `a·(B).sin = b·(A).sin` holds.
theorem point2_law_of_sines_sine_form(a: Point2[Real], b: Point2[Real], c: Point2[Real]) {
    b != a and c != a and c != b implies
    point2_norm(c.sub(b)) * sin_angle(a.sub(b), c.sub(b)) =
    point2_norm(c.sub(a)) * sin_angle(b.sub(a), c.sub(a))
} by {
    if b != a and c != a and c != b {
        if a.sub(b) = point2_zero[Real] {
            point2_sub_eq_zero_imp_eq[Real](a, b)
            a = b
            false
        }
        a.sub(b) != point2_zero[Real]
        if c.sub(b) = point2_zero[Real] {
            point2_sub_eq_zero_imp_eq[Real](c, b)
            c = b
            false
        }
        c.sub(b) != point2_zero[Real]
        if b.sub(a) = point2_zero[Real] {
            point2_sub_eq_zero_imp_eq[Real](b, a)
            b = a
            false
        }
        b.sub(a) != point2_zero[Real]
        if c.sub(a) = point2_zero[Real] {
            point2_sub_eq_zero_imp_eq[Real](c, a)
            c = a
            false
        }
        c.sub(a) != point2_zero[Real]
        point2_cross_eq_norm_mul_sin_angle(a.sub(b), c.sub(b))
        point2_norm(a.sub(b)) * point2_norm(c.sub(b)) * sin_angle(a.sub(b), c.sub(b)) =
            (a.sub(b).cross(c.sub(b))).abs
        point2_norm_pos(a.sub(b))
        Real.0 < point2_norm(a.sub(b))
        lt_imp_ne[Real](Real.0, point2_norm(a.sub(b)))
        point2_norm(a.sub(b)) != Real.0
        mul_left_cancel((a.sub(b).cross(c.sub(b))).abs, point2_norm(a.sub(b)),
            point2_norm(c.sub(b)) * sin_angle(a.sub(b), c.sub(b)))
        (a.sub(b).cross(c.sub(b))).abs / point2_norm(a.sub(b)) =
            point2_norm(c.sub(b)) * sin_angle(a.sub(b), c.sub(b))
        point2_norm(c.sub(b)) * sin_angle(a.sub(b), c.sub(b)) =
            (a.sub(b).cross(c.sub(b))).abs / point2_norm(a.sub(b))
        point2_cross_eq_norm_mul_sin_angle(b.sub(a), c.sub(a))
        point2_norm(b.sub(a)) * point2_norm(c.sub(a)) * sin_angle(b.sub(a), c.sub(a)) =
            (b.sub(a).cross(c.sub(a))).abs
        point2_norm_pos(b.sub(a))
        Real.0 < point2_norm(b.sub(a))
        lt_imp_ne[Real](Real.0, point2_norm(b.sub(a)))
        point2_norm(b.sub(a)) != Real.0
        mul_left_cancel((b.sub(a).cross(c.sub(a))).abs, point2_norm(b.sub(a)),
            point2_norm(c.sub(a)) * sin_angle(b.sub(a), c.sub(a)))
        (b.sub(a).cross(c.sub(a))).abs / point2_norm(b.sub(a)) =
            point2_norm(c.sub(a)) * sin_angle(b.sub(a), c.sub(a))
        point2_norm(c.sub(a)) * sin_angle(b.sub(a), c.sub(a)) =
            (b.sub(a).cross(c.sub(a))).abs / point2_norm(b.sub(a))
        point2_area2_cross_vertex_invariant(a, b, c)
        (c.sub(b).cross(a.sub(b))).abs = (b.sub(a).cross(c.sub(a))).abs
        point2_cross_swap(a.sub(b), c.sub(b))
        a.sub(b).cross(c.sub(b)) = -(c.sub(b).cross(a.sub(b)))
        abs_neg(c.sub(b).cross(a.sub(b)))
        (-(c.sub(b).cross(a.sub(b)))).abs = (c.sub(b).cross(a.sub(b))).abs
        (a.sub(b).cross(c.sub(b))).abs = (c.sub(b).cross(a.sub(b))).abs
        (a.sub(b).cross(c.sub(b))).abs = (b.sub(a).cross(c.sub(a))).abs
        point2_sub_reverse_neg(b, a)
        a.sub(b) = b.sub(a).neg
        point2_norm(a.sub(b)) = point2_norm(b.sub(a).neg)
        point2_norm_neg(b.sub(a))
        point2_norm(b.sub(a).neg) = point2_norm(b.sub(a))
        point2_norm(a.sub(b)) = point2_norm(b.sub(a))
        (b.sub(a).cross(c.sub(a))).abs / point2_norm(a.sub(b)) =
            (b.sub(a).cross(c.sub(a))).abs / point2_norm(b.sub(a))
        (a.sub(b).cross(c.sub(b))).abs / point2_norm(a.sub(b)) =
            (b.sub(a).cross(c.sub(a))).abs / point2_norm(b.sub(a))
        point2_norm(c.sub(b)) * sin_angle(a.sub(b), c.sub(b)) =
            point2_norm(c.sub(a)) * sin_angle(b.sub(a), c.sub(a))
    }
}

/// The sine of the angle between two nonzero, nonparallel vectors is nonzero.
theorem point2_sin_angle_ne_zero(u: Point2[Real], v: Point2[Real]) {
    u != point2_zero[Real] and v != point2_zero[Real] and u.cross(v) != Real.0 implies
    sin_angle(u, v) != Real.0
} by {
    if u != point2_zero[Real] and v != point2_zero[Real] and u.cross(v) != Real.0 {
        point2_norm_pos(u)
        Real.0 < point2_norm(u)
        lt_imp_ne[Real](Real.0, point2_norm(u))
        point2_norm(u) != Real.0
        point2_norm_pos(v)
        Real.0 < point2_norm(v)
        lt_imp_ne[Real](Real.0, point2_norm(v))
        point2_norm(v) != Real.0
        mul_not_zero[Real](point2_norm(u), point2_norm(v))
        point2_norm(u) * point2_norm(v) != Real.0
        if sin_angle(u, v) = Real.0 {
            sin_angle(u, v) = (u.cross(v)).abs / (point2_norm(u) * point2_norm(v))
            (u.cross(v)).abs / (point2_norm(u) * point2_norm(v)) = Real.0
            mul_div_cancel((u.cross(v)).abs, point2_norm(u) * point2_norm(v))
            point2_norm(u) * point2_norm(v) *
                ((u.cross(v)).abs / (point2_norm(u) * point2_norm(v))) = (u.cross(v)).abs
            point2_norm(u) * point2_norm(v) * Real.0 = Real.0
            point2_norm(u) * point2_norm(v) *
                ((u.cross(v)).abs / (point2_norm(u) * point2_norm(v))) =
                point2_norm(u) * point2_norm(v) * Real.0
            (u.cross(v)).abs = Real.0
            only_abs_zero_eq_zero(u.cross(v))
            u.cross(v) = Real.0
            false
        }
        sin_angle(u, v) != Real.0
    }
}

/// The law of sines: for a nondegenerate triangle, a side over the sine of
/// the opposite angle is the same for each side.  With `a = |b-c|` the side
/// opposite `A = ∠BAC` and `b = |c-a|` the side opposite `B = ∠CBA`,
/// `a / (A).sin = b / (B).sin`.
theorem point2_law_of_sines(a: Point2[Real], b: Point2[Real], c: Point2[Real]) {
    b != a and c != a and c != b and b.sub(a).cross(c.sub(a)) != Real.0 implies
    point2_norm(c.sub(b)) / sin_angle(b.sub(a), c.sub(a)) =
    point2_norm(c.sub(a)) / sin_angle(a.sub(b), c.sub(b))
} by {
    if b != a and c != a and c != b and b.sub(a).cross(c.sub(a)) != Real.0 {
        if b.sub(a) = point2_zero[Real] {
            point2_sub_eq_zero_imp_eq[Real](b, a)
            b = a
            false
        }
        b.sub(a) != point2_zero[Real]
        if c.sub(a) = point2_zero[Real] {
            point2_sub_eq_zero_imp_eq[Real](c, a)
            c = a
            false
        }
        c.sub(a) != point2_zero[Real]
        if a.sub(b) = point2_zero[Real] {
            point2_sub_eq_zero_imp_eq[Real](a, b)
            a = b
            false
        }
        a.sub(b) != point2_zero[Real]
        if c.sub(b) = point2_zero[Real] {
            point2_sub_eq_zero_imp_eq[Real](c, b)
            c = b
            false
        }
        c.sub(b) != point2_zero[Real]
        point2_cross_swap(a.sub(b), c.sub(b))
        a.sub(b).cross(c.sub(b)) = -(c.sub(b).cross(a.sub(b)))
        point2_cross_vertex_rotate(a, b, c)
        c.sub(b).cross(a.sub(b)) = b.sub(a).cross(c.sub(a))
        -(c.sub(b).cross(a.sub(b))) = -(b.sub(a).cross(c.sub(a)))
        a.sub(b).cross(c.sub(b)) = -(b.sub(a).cross(c.sub(a)))
        if a.sub(b).cross(c.sub(b)) = Real.0 {
            -(b.sub(a).cross(c.sub(a))) = Real.0
            -(-(b.sub(a).cross(c.sub(a)))) = -(Real.0)
            -(Real.0) = Real.0
            b.sub(a).cross(c.sub(a)) = Real.0
            false
        }
        a.sub(b).cross(c.sub(b)) != Real.0
        point2_law_of_sines_sine_form(a, b, c)
        point2_norm(c.sub(b)) * sin_angle(a.sub(b), c.sub(b)) =
            point2_norm(c.sub(a)) * sin_angle(b.sub(a), c.sub(a))
        point2_sin_angle_ne_zero(b.sub(a), c.sub(a))
        sin_angle(b.sub(a), c.sub(a)) != Real.0
        point2_sin_angle_ne_zero(a.sub(b), c.sub(b))
        sin_angle(a.sub(b), c.sub(b)) != Real.0
        prod_eq_to_div_eq(point2_norm(c.sub(b)), sin_angle(a.sub(b), c.sub(b)),
            point2_norm(c.sub(a)), sin_angle(b.sub(a), c.sub(a)))
        point2_norm(c.sub(b)) / sin_angle(b.sub(a), c.sub(a)) =
            point2_norm(c.sub(a)) / sin_angle(a.sub(b), c.sub(b))
    }
}
