from algebra.add_comm_group import AddCommGroup, sub_eq_zero_imp_eq, sub_add_cancel
from algebra.add_group import right_cancel, inverse_left
from algebra.add_ordered_group import negative_of_nonnegative, add_lt_add_right, lt_trans
from algebra.field.field import field_mul_eq_zero
from comm_ring import CommRing
from order import lte_antisymm
from ordered_field import OrderedField, zero_is_smaller_than_one
from geometry.point2 import Point2, point2_zero, point2_add_zero_right,
    point2_sub_self, point2_ext
from geometry.point2_algebra import point2_sub_add_sub, point2_sub_eq_add_neg,
    point2_dot_neg_right, point2_pythagoras_points_from_ab,
    point2_norm_sq_neg, point2_sub_reverse_neg, point2_dist_sq_comm,
    point2_norm_sq_sub_expansion, point2_norm_sq_add_expansion,
    point2_norm_sq_smul, point2_dist_sq_eq_norm_sq_sub, point2_sub_zero_right
from geometry.point2_affine import point2_add_sub_left_cancel,
    point2_sub_translate, point2_translate_translate, point2_smul_one_left
from geometry.point2_concyclic import concyclic4, concyclic4_of_equal_distances
from geometry.point2_heron import point2_ring_difference_of_squares

/// Nonnegative elements with equal squares are equal.
///
/// In an ordered field, `x^2 = y^2` factors as `(x - y) * (x + y) = 0`, so either
/// `x = y` or `x = -y`; the second alternative is excluded when both elements are
/// nonnegative.  This is what lets the length witnesses below be identified with
/// the squared distances they witness.
theorem ptolemy_nonneg_sq_eq[T: OrderedField](x: T, y: T) {
    T.0 <= x and T.0 <= y and x * x = y * y implies x = y
} by {
    if T.0 <= x and T.0 <= y and x * x = y * y {
        T.0 <= x
        T.0 <= y
        x * x = y * y
        point2_ring_difference_of_squares(x, y)
        (x + y) * (x - y) = x * x - y * y
        x * x - y * y = T.0
        (x + y) * (x - y) = T.0
        (x - y) * (x + y) = (x + y) * (x - y)
        (x - y) * (x + y) = T.0
        field_mul_eq_zero(x - y, x + y)
        x - y = T.0 or x + y = T.0
        if x - y = T.0 {
            sub_eq_zero_imp_eq(x, y)
            x = y
        } else {
            x + y = T.0
            inverse_left(y)
            -y + y = T.0
            x + y = -y + y
            right_cancel(x, -y, y)
            x = -y
            T.0 <= -y
            negative_of_nonnegative(-y)
            -(-y) <= T.0
            -(-y) = y
            y <= T.0
            lte_antisymm(y, T.0)
            y = T.0
            x = -T.0
            -T.0 = T.0
            x = T.0
            x = y
        }
    }
}

/// Subtracting the first summand of a sum gives the second summand.
theorem ptolemy_point_add_sub_right_cancel[T: AddCommGroup](p: Point2[T], q: Point2[T]) {
    p.add(q).sub(p) = q
} by {
    point2_add_zero_right(p)
    p.add(point2_zero[T]) = p
    point2_sub_add_sub(p, q, p, point2_zero[T])
    p.add(q).sub(p.add(point2_zero[T])) = p.sub(p).add(q.sub(point2_zero[T]))
    p.add(q).sub(p) = p.sub(p).add(q.sub(point2_zero[T]))
    point2_sub_self(p)
    p.sub(p) = point2_zero[T]
    p.add(q).sub(p) = point2_zero[T].add(q.sub(point2_zero[T]))
    point2_zero[T].add(q.sub(point2_zero[T])) = q
}

/// Subtracting two sums with a common base gives the difference of the summands.
theorem ptolemy_point_sub_add_same_base[T: AddCommGroup](a: Point2[T], u: Point2[T], v: Point2[T]) {
    a.add(u).sub(a.add(v)) = u.sub(v)
} by {
    point2_sub_add_sub(a, u, a, v)
    a.add(u).sub(a.add(v)) = a.sub(a).add(u.sub(v))
    point2_sub_self(a)
    a.sub(a) = point2_zero[T]
    a.add(u).sub(a.add(v)) = point2_zero[T].add(u.sub(v))
    point2_zero[T].add(u.sub(v)) = u.sub(v)
}

/// For orthogonal vectors, the sums and differences have equal squared norms.
///
/// `|u + v|^2 = |u|^2 + |v|^2 = |u - v|^2` whenever `u . v = 0`.
theorem point2_norm_sq_add_eq_sub_of_orthogonal[T: CommRing](u: Point2[T], v: Point2[T]) {
    u.orthogonal(v) implies u.add(v).norm_sq = u.sub(v).norm_sq
} by {
    if u.orthogonal(v) {
        u.dot(v) = T.0
        point2_norm_sq_add_expansion(u, v)
        u.add(v).norm_sq = u.norm_sq + v.norm_sq + u.dot(v) + u.dot(v)
        u.dot(v) = T.0
        u.add(v).norm_sq = u.norm_sq + v.norm_sq + T.0 + T.0
        u.add(v).norm_sq = u.norm_sq + v.norm_sq
        point2_sub_eq_add_neg(u, v)
        u.sub(v) = u.add(v.neg)
        point2_norm_sq_add_expansion(u, v.neg)
        u.add(v.neg).norm_sq = u.norm_sq + v.neg.norm_sq + u.dot(v.neg) + u.dot(v.neg)
        point2_norm_sq_neg(v)
        v.neg.norm_sq = v.norm_sq
        point2_dot_neg_right(u, v)
        u.dot(v.neg) = -u.dot(v)
        u.dot(v) = T.0
        u.dot(v.neg) = T.0
        u.add(v.neg).norm_sq = u.norm_sq + v.norm_sq + T.0 + T.0
        u.add(v.neg).norm_sq = u.norm_sq + v.norm_sq
        u.sub(v).norm_sq = u.norm_sq + v.norm_sq
        u.add(v).norm_sq = u.sub(v).norm_sq
    }
}

/// True when `a`, `b`, `c`, `d` are the corners of a rectangle in order.
///
/// The rectangle is a parallelogram — `c - a = (b - a) + (d - a)` — with a right
/// angle at `a`, so the two sides from `a` are orthogonal.  This is the same
/// encoding used for the British flag theorem.
define is_rectangle[T: OrderedField](a: Point2[T], b: Point2[T], c: Point2[T], d: Point2[T]) -> Bool {
    b.sub(a).orthogonal(d.sub(a)) and c = a.add(b.sub(a)).add(d.sub(a))
}

/// The two rectangle hypotheses: the right angle at `a` and the parallelogram law.
theorem is_rectangle_apply[T: OrderedField](a: Point2[T], b: Point2[T], c: Point2[T], d: Point2[T]) {
    is_rectangle(a, b, c, d) implies
        b.sub(a).orthogonal(d.sub(a)) and c = a.add(b.sub(a)).add(d.sub(a))
} by {
    if is_rectangle(a, b, c, d) {
        is_rectangle(a, b, c, d) =
            (b.sub(a).orthogonal(d.sub(a)) and c = a.add(b.sub(a)).add(d.sub(a)))
        (b.sub(a).orthogonal(d.sub(a)) and c = a.add(b.sub(a)).add(d.sub(a)))
    }
}

/// The diagonal `ac` of a rectangle satisfies the Pythagorean identity.
///
/// In a rectangle the diagonal squared is the sum of the squares of two adjacent
/// sides: `|c - a|^2 = |b - a|^2 + |c - b|^2`.  This is the reduced form of
/// Ptolemy's theorem for a rectangle, whose identity `AC * BD = AB * CD + BC * AD`
/// collapses to `AC^2 = AB^2 + BC^2` because the opposite sides and the diagonals
/// of a rectangle are equal.
theorem point2_rectangle_diagonal_pythagoras[T: OrderedField](
    a: Point2[T], b: Point2[T], c: Point2[T], d: Point2[T]
) {
    is_rectangle(a, b, c, d) implies
    c.sub(a).norm_sq = b.sub(a).norm_sq + c.sub(b).norm_sq
} by {
    if is_rectangle(a, b, c, d) {
        is_rectangle_apply(a, b, c, d)
        b.sub(a).orthogonal(d.sub(a))
        c = a.add(b.sub(a)).add(d.sub(a))
        point2_add_sub_left_cancel(a, b)
        b = a.add(b.sub(a))
        c = b.add(d.sub(a))
        ptolemy_point_add_sub_right_cancel(b, d.sub(a))
        c.sub(b) = d.sub(a)
        b.sub(a).orthogonal(c.sub(b))
        point2_pythagoras_points_from_ab(a, b, c)
        c.sub(a).norm_sq = b.sub(a).norm_sq + c.sub(b).norm_sq
    }
}

/// In a rectangle the diagonal `ac` squared is the sum of the squares of `ab` and `bc`.
///
/// With the length witnesses of the general statement this is the content of
/// Ptolemy's theorem specialized to a rectangle.
theorem ptolemy_rectangle_squared[T: OrderedField](
    a: Point2[T], b: Point2[T], c: Point2[T], d: Point2[T]
) {
    is_rectangle(a, b, c, d) implies
    a.dist_sq(c) = a.dist_sq(b) + b.dist_sq(c)
} by {
    if is_rectangle(a, b, c, d) {
        point2_rectangle_diagonal_pythagoras(a, b, c, d)
        c.sub(a).norm_sq = b.sub(a).norm_sq + c.sub(b).norm_sq
        point2_dist_sq_comm(a, c)
        a.dist_sq(c) = c.dist_sq(a)
        c.dist_sq(a) = c.sub(a).norm_sq
        a.dist_sq(c) = c.sub(a).norm_sq
        point2_dist_sq_comm(a, b)
        a.dist_sq(b) = b.dist_sq(a)
        b.dist_sq(a) = b.sub(a).norm_sq
        a.dist_sq(b) = b.sub(a).norm_sq
        point2_dist_sq_comm(b, c)
        b.dist_sq(c) = c.dist_sq(b)
        c.dist_sq(b) = c.sub(b).norm_sq
        b.dist_sq(c) = c.sub(b).norm_sq
        a.dist_sq(c) = a.dist_sq(b) + b.dist_sq(c)
    }
}

/// The opposite sides of a rectangle have equal squared lengths.
theorem point2_rectangle_opposite_sides_dist_sq_eq[T: OrderedField](
    a: Point2[T], b: Point2[T], c: Point2[T], d: Point2[T]
) {
    is_rectangle(a, b, c, d) implies
    c.dist_sq(d) = a.dist_sq(b) and d.dist_sq(a) = b.dist_sq(c)
} by {
    if is_rectangle(a, b, c, d) {
        is_rectangle_apply(a, b, c, d)
        b.sub(a).orthogonal(d.sub(a))
        c = a.add(b.sub(a)).add(d.sub(a))
        point2_add_sub_left_cancel(a, b)
        b = a.add(b.sub(a))
        point2_add_sub_left_cancel(a, d)
        d = a.add(d.sub(a))
        c = b.add(d.sub(a))
        // the displacement from c to d equals the displacement from b to a
        point2_sub_translate(b, a, d.sub(a))
        c.sub(d) = b.sub(a)
        point2_sub_reverse_neg(a, b)
        a.sub(b) = b.sub(a).neg
        point2_norm_sq_neg(b.sub(a))
        b.sub(a).neg.norm_sq = b.sub(a).norm_sq
        c.dist_sq(d) = a.dist_sq(b)
        // the displacement from d to a equals the negative of the displacement from b to c
        ptolemy_point_add_sub_right_cancel(b, d.sub(a))
        c.sub(b) = d.sub(a)
        point2_sub_reverse_neg(b, c)
        b.sub(c) = c.sub(b).neg
        point2_norm_sq_neg(d.sub(a))
        d.sub(a).neg.norm_sq = d.sub(a).norm_sq
        d.dist_sq(a) = b.dist_sq(c)
        c.dist_sq(d) = a.dist_sq(b) and d.dist_sq(a) = b.dist_sq(c)
    }
}

/// The diagonals of a rectangle have equal squared lengths.
theorem point2_rectangle_diagonals_dist_sq_eq[T: OrderedField](
    a: Point2[T], b: Point2[T], c: Point2[T], d: Point2[T]
) {
    is_rectangle(a, b, c, d) implies a.dist_sq(c) = b.dist_sq(d)
} by {
    if is_rectangle(a, b, c, d) {
        is_rectangle_apply(a, b, c, d)
        b.sub(a).orthogonal(d.sub(a))
        c = a.add(b.sub(a)).add(d.sub(a))
        point2_add_sub_left_cancel(a, b)
        b = a.add(b.sub(a))
        point2_add_sub_left_cancel(a, d)
        d = a.add(d.sub(a))
        // c - a = (b - a) + (d - a)
        point2_translate_translate(a, b.sub(a), d.sub(a))
        a.add(b.sub(a)).add(d.sub(a)) = a.add(b.sub(a).add(d.sub(a)))
        c = a.add(b.sub(a).add(d.sub(a)))
        ptolemy_point_add_sub_right_cancel(a, b.sub(a).add(d.sub(a)))
        c.sub(a) = b.sub(a).add(d.sub(a))
        // b - d = (b - a) - (d - a)
        ptolemy_point_sub_add_same_base(a, b.sub(a), d.sub(a))
        b.sub(d) = b.sub(a).sub(d.sub(a))
        // |u + v|^2 = |u - v|^2
        point2_norm_sq_add_eq_sub_of_orthogonal(b.sub(a), d.sub(a))
        b.sub(a).add(d.sub(a)).norm_sq = b.sub(a).sub(d.sub(a)).norm_sq
        c.sub(a).norm_sq = b.sub(a).add(d.sub(a)).norm_sq
        b.sub(d).norm_sq = b.sub(a).sub(d.sub(a)).norm_sq
        c.sub(a).norm_sq = b.sub(d).norm_sq
        point2_dist_sq_comm(a, c)
        a.dist_sq(c) = c.dist_sq(a)
        c.dist_sq(a) = c.sub(a).norm_sq
        a.dist_sq(c) = c.sub(a).norm_sq
        b.dist_sq(d) = b.sub(d).norm_sq
        a.dist_sq(c) = b.dist_sq(d)
    }
}

/// Ptolemy's theorem for a rectangle, with side and diagonal length witnesses.
///
/// In a rectangle the opposite sides and the diagonals are equal, so Ptolemy's
/// identity `AC * BD = AB * CD + BC * AD` collapses to the Pythagorean identity
/// `AC^2 = AB^2 + BC^2`.  With the length witnesses of the general statement
/// (`w^2 = AB^2`, `x^2 = BC^2`, `y^2 = CD^2`, `z^2 = DA^2`, `p^2 = AC^2`,
/// `q^2 = BD^2`, all nonnegative) the conclusion `p * q = w * y + x * z` follows
/// from the rectangle's distance equalities and the nonnegativity of the
/// witnesses, with no angle machinery.
theorem ptolemy_rectangle[T: OrderedField](
    a: Point2[T], b: Point2[T], c: Point2[T], d: Point2[T],
    w: T, x: T, y: T, z: T, p: T, q: T
) {
    is_rectangle(a, b, c, d) and
    w * w = a.dist_sq(b) and x * x = b.dist_sq(c) and
    y * y = c.dist_sq(d) and z * z = d.dist_sq(a) and
    p * p = a.dist_sq(c) and q * q = b.dist_sq(d) and
    w >= T.0 and x >= T.0 and y >= T.0 and z >= T.0 and p >= T.0 and q >= T.0
    implies p * q = w * y + x * z
} by {
    if is_rectangle(a, b, c, d) and
        w * w = a.dist_sq(b) and x * x = b.dist_sq(c) and
        y * y = c.dist_sq(d) and z * z = d.dist_sq(a) and
        p * p = a.dist_sq(c) and q * q = b.dist_sq(d) and
        w >= T.0 and x >= T.0 and y >= T.0 and z >= T.0 and p >= T.0 and q >= T.0 {
        is_rectangle(a, b, c, d)
        w * w = a.dist_sq(b)
        x * x = b.dist_sq(c)
        y * y = c.dist_sq(d)
        z * z = d.dist_sq(a)
        p * p = a.dist_sq(c)
        q * q = b.dist_sq(d)
        w >= T.0
        x >= T.0
        y >= T.0
        z >= T.0
        p >= T.0
        q >= T.0
        // the diagonal is the hypotenuse of the right triangle with legs `ab` and `bc`
        ptolemy_rectangle_squared(a, b, c, d)
        a.dist_sq(c) = a.dist_sq(b) + b.dist_sq(c)
        // opposite sides and diagonals are equal
        point2_rectangle_opposite_sides_dist_sq_eq(a, b, c, d)
        c.dist_sq(d) = a.dist_sq(b)
        d.dist_sq(a) = b.dist_sq(c)
        point2_rectangle_diagonals_dist_sq_eq(a, b, c, d)
        a.dist_sq(c) = b.dist_sq(d)
        // witnesses identify the equal lengths
        p * p = q * q
        T.0 <= p
        T.0 <= q
        T.0 <= p and T.0 <= q and p * p = q * q
        ptolemy_nonneg_sq_eq(p, q)
        p = q
        w * w = y * y
        T.0 <= w
        T.0 <= y
        T.0 <= w and T.0 <= y and w * w = y * y
        ptolemy_nonneg_sq_eq(w, y)
        w = y
        x * x = z * z
        T.0 <= x
        T.0 <= z
        T.0 <= x and T.0 <= z and x * x = z * z
        ptolemy_nonneg_sq_eq(x, z)
        x = z
        // p * q = p^2 = AC^2 = AB^2 + BC^2 = w^2 + x^2 = w * y + x * z
        p * q = p * p
        p * p = a.dist_sq(c)
        a.dist_sq(c) = a.dist_sq(b) + b.dist_sq(c)
        a.dist_sq(b) = w * w
        b.dist_sq(c) = x * x
        a.dist_sq(c) = w * w + x * x
        p * p = w * w + x * x
        w * y = w * w
        x * z = x * x
        w * y + x * z = w * w + x * x
        p * q = w * y + x * z
    }
}

/// Scalar multiplication distributes over point addition.
theorem ptolemy_smul_add[T: CommRing](scalar: T, p: Point2[T], q: Point2[T]) {
    p.add(q).smul(scalar) = p.smul(scalar).add(q.smul(scalar))
} by {
    let lhs = p.add(q).smul(scalar)
    let rhs = p.smul(scalar).add(q.smul(scalar))
    lhs.x = (p.x + q.x) * scalar
    rhs.x = p.x * scalar + q.x * scalar
    (p.x + q.x) * scalar = p.x * scalar + q.x * scalar
    lhs.x = rhs.x
    lhs.y = (p.y + q.y) * scalar
    rhs.y = p.y * scalar + q.y * scalar
    (p.y + q.y) * scalar = p.y * scalar + q.y * scalar
    lhs.y = rhs.y
    point2_ext(lhs, rhs)
}

/// Scalar multiplication distributes over point subtraction.
theorem ptolemy_smul_sub[T: CommRing](scalar: T, p: Point2[T], q: Point2[T]) {
    p.sub(q).smul(scalar) = p.smul(scalar).sub(q.smul(scalar))
} by {
    let lhs = p.sub(q).smul(scalar)
    let rhs = p.smul(scalar).sub(q.smul(scalar))
    lhs.x = (p.x - q.x) * scalar
    rhs.x = p.x * scalar - q.x * scalar
    (p.x - q.x) * scalar = p.x * scalar - q.x * scalar
    lhs.x = rhs.x
    lhs.y = (p.y - q.y) * scalar
    rhs.y = p.y * scalar - q.y * scalar
    (p.y - q.y) * scalar = p.y * scalar - q.y * scalar
    lhs.y = rhs.y
    point2_ext(lhs, rhs)
}

/// Scalar multiplication distributes over scalar subtraction.
theorem ptolemy_smul_scalar_sub[T: CommRing](a: T, b: T, p: Point2[T]) {
    p.smul(a).sub(p.smul(b)) = p.smul(a - b)
} by {
    let lhs = p.smul(a).sub(p.smul(b))
    let rhs = p.smul(a - b)
    lhs.x = p.x * a - p.x * b
    rhs.x = p.x * (a - b)
    p.x * a - p.x * b = p.x * (a - b)
    lhs.x = rhs.x
    lhs.y = p.y * a - p.y * b
    rhs.y = p.y * (a - b)
    p.y * a - p.y * b = p.y * (a - b)
    lhs.y = rhs.y
    point2_ext(lhs, rhs)
}

/// One plus one is not zero in an ordered field.
theorem ptolemy_one_plus_one_neq_zero[T: OrderedField] {
    T.1 + T.1 != T.0
} by {
    zero_is_smaller_than_one[T]
    T.0 < T.1
    add_lt_add_right(T.0, T.1, T.1)
    T.0 + T.1 < T.1 + T.1
    T.0 + T.1 = T.1
    T.1 < T.1 + T.1
    lt_trans(T.0, T.1, T.1 + T.1)
    T.0 < T.1 + T.1
    T.1 + T.1 != T.0
}

/// The inverse of two plus itself is one: `1/2 + 1/2 = 1`.
theorem ptolemy_half_plus_half[T: OrderedField] {
    (T.1 + T.1).inverse + (T.1 + T.1).inverse = T.1
} by {
    ptolemy_one_plus_one_neq_zero[T]
    T.1 + T.1 != T.0
    (T.1 + T.1) * (T.1 + T.1).inverse = T.1
    (T.1 + T.1) * (T.1 + T.1).inverse =
        (T.1 + T.1).inverse + (T.1 + T.1).inverse
    (T.1 + T.1).inverse + (T.1 + T.1).inverse = T.1
}

/// The four corners of a rectangle lie on a common circle.
///
/// The circumcenter is the midpoint of a diagonal: the point
/// `m = a + (b - a + d - a) * h` with `h = (1 + 1).inverse` is equidistant from
/// the four corners, because the two side vectors from `a` are orthogonal, so
/// `|(b - a) + (d - a)| = |(b - a) - (d - a)|`.
theorem point2_rectangle_concyclic[T: OrderedField](
    a: Point2[T], b: Point2[T], c: Point2[T], d: Point2[T]
) {
    is_rectangle(a, b, c, d) implies concyclic4(a, b, c, d)
} by {
    if is_rectangle(a, b, c, d) {
        let h: T = (T.1 + T.1).inverse
        let m: Point2[T] = a.add(b.sub(a).add(d.sub(a)).smul(h))
        is_rectangle_apply(a, b, c, d)
        b.sub(a).orthogonal(d.sub(a))
        c = a.add(b.sub(a)).add(d.sub(a))
        point2_add_sub_left_cancel(a, b)
        b = a.add(b.sub(a))
        point2_add_sub_left_cancel(a, d)
        d = a.add(d.sub(a))
        ptolemy_half_plus_half[T]
        h + h = T.1
        sub_add_cancel(T.1, h)
        (T.1 - h) + h = T.1
        (T.1 - h) + h = h + h
        right_cancel(T.1 - h, h, h)
        T.1 - h = h
        // m - a = (b - a + d - a) * h
        ptolemy_point_add_sub_right_cancel(a, b.sub(a).add(d.sub(a)).smul(h))
        m.sub(a) = b.sub(a).add(d.sub(a)).smul(h)
        // a - m = -(m - a)
        point2_sub_reverse_neg(a, m)
        a.sub(m) = m.sub(a).neg
        // c - m = (m - a), since c = a + (b - a + d - a)
        point2_translate_translate(a, b.sub(a), d.sub(a))
        a.add(b.sub(a)).add(d.sub(a)) = a.add(b.sub(a).add(d.sub(a)))
        c = a.add(b.sub(a).add(d.sub(a)))
        ptolemy_point_sub_add_same_base(a, b.sub(a).add(d.sub(a)), b.sub(a).add(d.sub(a)).smul(h))
        c.sub(m) = b.sub(a).add(d.sub(a)).sub(b.sub(a).add(d.sub(a)).smul(h))
        point2_smul_one_left(b.sub(a).add(d.sub(a)))
        b.sub(a).add(d.sub(a)).smul(T.1) = b.sub(a).add(d.sub(a))
        ptolemy_smul_scalar_sub(T.1, h, b.sub(a).add(d.sub(a)))
        b.sub(a).add(d.sub(a)).smul(T.1).sub(b.sub(a).add(d.sub(a)).smul(h)) =
            b.sub(a).add(d.sub(a)).smul(T.1 - h)
        c.sub(m) = b.sub(a).add(d.sub(a)).smul(T.1 - h)
        c.sub(m) = b.sub(a).add(d.sub(a)).smul(h)
        c.sub(m) = m.sub(a)
        // |a - m|^2 = |c - m|^2
        a.dist_sq(m) = a.sub(m).norm_sq
        point2_norm_sq_neg(m.sub(a))
        m.sub(a).neg.norm_sq = m.sub(a).norm_sq
        a.sub(m).norm_sq = m.sub(a).norm_sq
        a.dist_sq(m) = m.sub(a).norm_sq
        c.dist_sq(m) = c.sub(m).norm_sq
        c.sub(m) = m.sub(a)
        c.dist_sq(m) = m.sub(a).norm_sq
        a.dist_sq(m) = c.dist_sq(m)
        // b - m = (b - a - (d - a)) * h
        ptolemy_point_sub_add_same_base(a, b.sub(a), b.sub(a).add(d.sub(a)).smul(h))
        b.sub(m) = b.sub(a).sub(b.sub(a).add(d.sub(a)).smul(h))
        ptolemy_smul_add(h, b.sub(a), d.sub(a))
        b.sub(a).add(d.sub(a)).smul(h) = b.sub(a).smul(h).add(d.sub(a).smul(h))
        b.sub(m) = b.sub(a).sub(b.sub(a).smul(h).add(d.sub(a).smul(h)))
        point2_add_zero_right(b.sub(a))
        b.sub(a).add(point2_zero[T]) = b.sub(a)
        point2_sub_add_sub(b.sub(a), point2_zero[T], b.sub(a).smul(h), d.sub(a).smul(h))
        b.sub(a).add(point2_zero[T]).sub(b.sub(a).smul(h).add(d.sub(a).smul(h))) =
            b.sub(a).sub(b.sub(a).smul(h)).add(point2_zero[T].sub(d.sub(a).smul(h)))
        b.sub(m) = b.sub(a).sub(b.sub(a).smul(h)).add(point2_zero[T].sub(d.sub(a).smul(h)))
        point2_smul_one_left(b.sub(a))
        b.sub(a).smul(T.1) = b.sub(a)
        ptolemy_smul_scalar_sub(T.1, h, b.sub(a))
        b.sub(a).smul(T.1).sub(b.sub(a).smul(h)) = b.sub(a).smul(T.1 - h)
        b.sub(a).sub(b.sub(a).smul(h)) = b.sub(a).smul(T.1 - h)
        b.sub(a).sub(b.sub(a).smul(h)) = b.sub(a).smul(h)
        point2_sub_zero_right(d.sub(a).smul(h))
        d.sub(a).smul(h).sub(point2_zero[T]) = d.sub(a).smul(h)
        point2_sub_reverse_neg(point2_zero[T], d.sub(a).smul(h))
        point2_zero[T].sub(d.sub(a).smul(h)) = d.sub(a).smul(h).sub(point2_zero[T]).neg
        point2_zero[T].sub(d.sub(a).smul(h)) = d.sub(a).smul(h).neg
        b.sub(m) = b.sub(a).smul(h).add(d.sub(a).smul(h).neg)
        point2_sub_eq_add_neg(b.sub(a).smul(h), d.sub(a).smul(h))
        b.sub(a).smul(h).sub(d.sub(a).smul(h)) = b.sub(a).smul(h).add(d.sub(a).smul(h).neg)
        b.sub(m) = b.sub(a).smul(h).sub(d.sub(a).smul(h))
        ptolemy_smul_sub(h, b.sub(a), d.sub(a))
        b.sub(a).sub(d.sub(a)).smul(h) = b.sub(a).smul(h).sub(d.sub(a).smul(h))
        b.sub(m) = b.sub(a).sub(d.sub(a)).smul(h)
        // |b - m|^2 = h^2 * |u - v|^2 = h^2 * |u + v|^2 = |a - m|^2
        point2_norm_sq_smul(h, b.sub(a).sub(d.sub(a)))
        b.sub(a).sub(d.sub(a)).smul(h).norm_sq = h * h * b.sub(a).sub(d.sub(a)).norm_sq
        b.sub(m).norm_sq = h * h * b.sub(a).sub(d.sub(a)).norm_sq
        b.dist_sq(m) = b.sub(m).norm_sq
        b.dist_sq(m) = h * h * b.sub(a).sub(d.sub(a)).norm_sq
        point2_norm_sq_add_eq_sub_of_orthogonal(b.sub(a), d.sub(a))
        b.sub(a).add(d.sub(a)).norm_sq = b.sub(a).sub(d.sub(a)).norm_sq
        point2_norm_sq_smul(h, b.sub(a).add(d.sub(a)))
        b.sub(a).add(d.sub(a)).smul(h).norm_sq = h * h * b.sub(a).add(d.sub(a)).norm_sq
        m.sub(a).norm_sq = h * h * b.sub(a).add(d.sub(a)).norm_sq
        m.sub(a).norm_sq = h * h * b.sub(a).sub(d.sub(a)).norm_sq
        a.dist_sq(m) = m.sub(a).norm_sq
        a.dist_sq(m) = h * h * b.sub(a).sub(d.sub(a)).norm_sq
        b.dist_sq(m) = a.dist_sq(m)
        // d - m = (d - a - (b - a)) * h
        ptolemy_point_sub_add_same_base(a, d.sub(a), b.sub(a).add(d.sub(a)).smul(h))
        d.sub(m) = d.sub(a).sub(b.sub(a).add(d.sub(a)).smul(h))
        ptolemy_smul_add(h, b.sub(a), d.sub(a))
        b.sub(a).add(d.sub(a)).smul(h) = b.sub(a).smul(h).add(d.sub(a).smul(h))
        d.sub(m) = d.sub(a).sub(b.sub(a).smul(h).add(d.sub(a).smul(h)))
        point2_add_zero_right(d.sub(a))
        d.sub(a).add(point2_zero[T]) = d.sub(a)
        point2_sub_add_sub(d.sub(a), point2_zero[T], b.sub(a).smul(h), d.sub(a).smul(h))
        d.sub(a).add(point2_zero[T]).sub(b.sub(a).smul(h).add(d.sub(a).smul(h))) =
            d.sub(a).sub(d.sub(a).smul(h)).add(point2_zero[T].sub(b.sub(a).smul(h)))
        d.sub(m) = d.sub(a).sub(d.sub(a).smul(h)).add(point2_zero[T].sub(b.sub(a).smul(h)))
        point2_smul_one_left(d.sub(a))
        d.sub(a).smul(T.1) = d.sub(a)
        ptolemy_smul_scalar_sub(T.1, h, d.sub(a))
        d.sub(a).smul(T.1).sub(d.sub(a).smul(h)) = d.sub(a).smul(T.1 - h)
        d.sub(a).sub(d.sub(a).smul(h)) = d.sub(a).smul(T.1 - h)
        d.sub(a).sub(d.sub(a).smul(h)) = d.sub(a).smul(h)
        point2_sub_zero_right(b.sub(a).smul(h))
        b.sub(a).smul(h).sub(point2_zero[T]) = b.sub(a).smul(h)
        point2_sub_reverse_neg(point2_zero[T], b.sub(a).smul(h))
        point2_zero[T].sub(b.sub(a).smul(h)) = b.sub(a).smul(h).sub(point2_zero[T]).neg
        point2_zero[T].sub(b.sub(a).smul(h)) = b.sub(a).smul(h).neg
        d.sub(m) = d.sub(a).smul(h).add(b.sub(a).smul(h).neg)
        point2_sub_eq_add_neg(d.sub(a).smul(h), b.sub(a).smul(h))
        d.sub(a).smul(h).sub(b.sub(a).smul(h)) = d.sub(a).smul(h).add(b.sub(a).smul(h).neg)
        d.sub(m) = d.sub(a).smul(h).sub(b.sub(a).smul(h))
        ptolemy_smul_sub(h, d.sub(a), b.sub(a))
        d.sub(a).sub(b.sub(a)).smul(h) = d.sub(a).smul(h).sub(b.sub(a).smul(h))
        d.sub(m) = d.sub(a).sub(b.sub(a)).smul(h)
        // |d - m|^2 = h^2 * |v - u|^2 = h^2 * |u - v|^2 = |b - m|^2
        point2_norm_sq_smul(h, d.sub(a).sub(b.sub(a)))
        d.sub(a).sub(b.sub(a)).smul(h).norm_sq = h * h * d.sub(a).sub(b.sub(a)).norm_sq
        d.sub(m).norm_sq = h * h * d.sub(a).sub(b.sub(a)).norm_sq
        d.dist_sq(m) = d.sub(m).norm_sq
        d.dist_sq(m) = h * h * d.sub(a).sub(b.sub(a)).norm_sq
        point2_sub_reverse_neg(b.sub(a), d.sub(a))
        b.sub(a).sub(d.sub(a)) = d.sub(a).sub(b.sub(a)).neg
        point2_norm_sq_neg(d.sub(a).sub(b.sub(a)))
        d.sub(a).sub(b.sub(a)).neg.norm_sq = d.sub(a).sub(b.sub(a)).norm_sq
        b.sub(a).sub(d.sub(a)).norm_sq = d.sub(a).sub(b.sub(a)).norm_sq
        b.dist_sq(m) = h * h * b.sub(a).sub(d.sub(a)).norm_sq
        b.dist_sq(m) = h * h * d.sub(a).sub(b.sub(a)).norm_sq
        d.dist_sq(m) = b.dist_sq(m)
        a.dist_sq(m) = d.dist_sq(m)
        a.dist_sq(m) = c.dist_sq(m) and a.dist_sq(m) = b.dist_sq(m) and a.dist_sq(m) = d.dist_sq(m)
        concyclic4_of_equal_distances(m, a, b, c, d)
        concyclic4(a, b, c, d)
    }
}

// Ptolemy's theorem: for a cyclic quadrilateral `a`, `b`, `c`, `d` (in order),
// the product of the diagonals equals the sum of the products of the opposite
// sides: `AC * BD = AB * CD + BC * AD`.
//
// The library has no distance function, only squared distances, so the lengths
// are introduced as nonnegative witnesses, as in Heron's theorem
// (`point2_heron_triangle_squared`): `w^2 = |b - a|^2`, `x^2 = |c - b|^2`,
// `y^2 = |d - c|^2`, `z^2 = |a - d|^2` for the sides and `p^2 = |c - a|^2`,
// `q^2 = |d - b|^2` for the diagonals, with the conclusion `p * q = w * y + x * z`.
// The cyclicity hypothesis is the library's `concyclic4` predicate: the four
// points lie on a common circle.  Unlike Heron's product, the identity is not
// invariant under changing the sign of a single witness, so the witnesses are
// required to be nonnegative; without that the statement is false.
//
// The classical proof splits the diagonal `ac` at the point where the inscribed
// angles subtending the pieces are equal, producing two pairs of similar
// triangles (`abe ~ dce` and `bce ~ ade`) whose side ratios multiply to the
// identity.  This needs the equality of inscribed angles subtending a chord,
// which the library does not yet have, so the proof is left for future angle
// machinery; see also `theorems1000.theorem_ptolemy` for the same statement in
// the r2 (Pair[Real, Real]) form.  The rectangle special case above is proved:
// in a rectangle the identity reduces to the Pythagorean theorem, and
// `ptolemy_rectangle` derives the full witness form `p * q = w * y + x * z`
// from the rectangle hypotheses alone.
//
// theorem ptolemy_cyclic_quadrilateral[T: OrderedField](
//     a: Point2[T], b: Point2[T], c: Point2[T], d: Point2[T],
//     w: T, x: T, y: T, z: T, p: T, q: T
// ) {
//     concyclic4(a, b, c, d) and
//     w * w = a.dist_sq(b) and x * x = b.dist_sq(c) and
//     y * y = c.dist_sq(d) and z * z = d.dist_sq(a) and
//     p * p = a.dist_sq(c) and q * q = b.dist_sq(d) and
//     w >= T.0 and x >= T.0 and y >= T.0 and z >= T.0 and p >= T.0 and q >= T.0
//     implies p * q = w * y + x * z
// }
