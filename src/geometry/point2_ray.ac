from ordered_field import OrderedField
from geometry.point2 import Point2
from geometry.point2_affine import point2_param_line_zero,
    point2_param_line_one, point2_param_line_collinear,
    point2_param_line_translate, point2_param_line_reverse,
    point2_translate_neg_right
from geometry.point2_segment import point2_between_iff_on_segment,
    point2_on_segment_translate_forward

attributes Point2[T: OrderedField] {
    /// True when `p` lies on the ray starting at `self` and passing through `b`.
    define on_ray(self, b: Point2[T], p: Point2[T]) -> Bool {
        exists(t: T) {
            T.0 <= t and p = self.param_line(b, t)
        }
    }
}

/// The starting point lies on the ray.
theorem point2_on_ray_start[T: OrderedField](a: Point2[T], b: Point2[T]) {
    a.on_ray(b, a)
} by {
    point2_param_line_zero(a, b)
    exists(t: T) {
        T.0 <= t and a = a.param_line(b, t)
    }
}

/// The second defining point lies on the ray.
theorem point2_on_ray_through[T: OrderedField](a: Point2[T], b: Point2[T]) {
    a.on_ray(b, b)
} by {
    point2_param_line_one(a, b)
    T.0 <= T.1
}

/// A parameter-line point with nonnegative parameter lies on the ray.
theorem point2_param_line_on_ray[T: OrderedField](a: Point2[T], b: Point2[T], t: T) {
    T.0 <= t implies a.on_ray(b, a.param_line(b, t))
} by {
    if T.0 <= t {
        exists(s: T) {
            T.0 <= s and a.param_line(b, t) = a.param_line(b, s)
        }
    }
}

/// Ray membership implies line membership.
theorem point2_on_ray_imp_on_line[T: OrderedField](a: Point2[T], b: Point2[T], p: Point2[T]) {
    a.on_ray(b, p) implies a.on_line(b, p)
} by {
    if a.on_ray(b, p) {
        let t: T satisfy {
            T.0 <= t and p = a.param_line(b, t)
        }
        point2_param_line_collinear(a, b, t)
        a.collinear(b, p)
    }
}

/// A parameter-line point with parameter at most one lies on the reverse ray.
theorem point2_param_line_on_reverse_ray_of_le_one[T: OrderedField](a: Point2[T], b: Point2[T], t: T) {
    t <= T.1 implies b.on_ray(a, a.param_line(b, t))
} by {
    if t <= T.1 {
        let s = T.1 - t
        T.0 <= T.1 - t
        point2_param_line_reverse(a, b, t)
        exists(u: T) {
            T.0 <= u and a.param_line(b, t) = b.param_line(a, u)
        }
    }
}

/// Segment membership implies ray membership from the same start point.
theorem point2_on_segment_imp_on_ray[T: OrderedField](a: Point2[T], b: Point2[T], p: Point2[T]) {
    a.on_segment(b, p) implies a.on_ray(b, p)
} by {
    if a.on_segment(b, p) {
        let t: T satisfy {
            T.0 <= t and t <= T.1 and p = a.param_line(b, t)
        }
        exists(s: T) {
            T.0 <= s and p = a.param_line(b, s)
        }
    }
}

/// Betweenness implies ray membership from the first endpoint.
theorem point2_between_imp_on_ray[T: OrderedField](a: Point2[T], p: Point2[T], b: Point2[T]) {
    a.between(p, b) implies a.on_ray(b, p)
} by {
    if a.between(p, b) {
        point2_on_segment_imp_on_ray(a, b, p)
    }
}

/// Segment membership implies ray membership from the opposite endpoint.
theorem point2_on_segment_imp_on_reverse_ray[T: OrderedField](a: Point2[T], b: Point2[T], p: Point2[T]) {
    a.on_segment(b, p) implies b.on_ray(a, p)
} by {
    if a.on_segment(b, p) {
        let t: T satisfy {
            T.0 <= t and t <= T.1 and p = a.param_line(b, t)
        }
        point2_param_line_on_reverse_ray_of_le_one(a, b, t)
        b.on_ray(a, p)
    }
}

/// Betweenness implies ray membership from the second endpoint.
theorem point2_between_imp_on_reverse_ray[T: OrderedField](a: Point2[T], p: Point2[T], b: Point2[T]) {
    a.between(p, b) implies b.on_ray(a, p)
} by {
    if a.between(p, b) {
        point2_between_iff_on_segment(a, p, b)
        a.on_segment(b, p)
        point2_on_segment_imp_on_reverse_ray(a, b, p)
    }
}

/// Translating a point on a ray gives a point on the translated ray.
theorem point2_on_ray_translate_forward[T: OrderedField](a: Point2[T], b: Point2[T], p: Point2[T], v: Point2[T]) {
    a.on_ray(b, p) implies a.translate(v).on_ray(b.translate(v), p.translate(v))
} by {
    if a.on_ray(b, p) {
        let t: T satisfy {
            T.0 <= t and p = a.param_line(b, t)
        }
        point2_param_line_translate(a, b, t, v)
        exists(s: T) {
            T.0 <= s and p.translate(v) = a.translate(v).param_line(b.translate(v), s)
        }
    }
}

/// Translating all points preserves ray membership.
theorem point2_on_ray_translate[T: OrderedField](a: Point2[T], b: Point2[T], p: Point2[T], v: Point2[T]) {
    a.translate(v).on_ray(b.translate(v), p.translate(v)) = a.on_ray(b, p)
} by {
    if a.on_ray(b, p) {
        point2_on_ray_translate_forward(a, b, p, v)
    }
    if a.translate(v).on_ray(b.translate(v), p.translate(v)) {
        point2_on_ray_translate_forward(a.translate(v), b.translate(v), p.translate(v), v.neg)
        point2_translate_neg_right(a, v)
        point2_translate_neg_right(b, v)
        point2_translate_neg_right(p, v)
        a.on_ray(b, p)
    }
}

/// A point on a translated segment lies on the corresponding translated ray.
theorem point2_on_segment_translate_imp_on_ray[T: OrderedField](a: Point2[T], b: Point2[T], p: Point2[T], v: Point2[T]) {
    a.on_segment(b, p) implies a.translate(v).on_ray(b.translate(v), p.translate(v))
} by {
    if a.on_segment(b, p) {
        point2_on_segment_translate_forward(a, b, p, v)
        point2_on_segment_imp_on_ray(a.translate(v), b.translate(v), p.translate(v))
    }
}

/// A point on a segment translates to a point on the translated reverse ray.
theorem point2_on_segment_translate_imp_on_reverse_ray[T: OrderedField](a: Point2[T], b: Point2[T], p: Point2[T], v: Point2[T]) {
    a.on_segment(b, p) implies b.translate(v).on_ray(a.translate(v), p.translate(v))
} by {
    if a.on_segment(b, p) {
        point2_on_segment_imp_on_reverse_ray(a, b, p)
        point2_on_ray_translate_forward(b, a, p, v)
        b.translate(v).on_ray(a.translate(v), p.translate(v))
    }
}
