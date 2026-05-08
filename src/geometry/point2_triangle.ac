from ordered_field import OrderedField
from geometry.point2 import Point2, point2_orientation_same_first,
    point2_orientation_same_second, point2_orientation_same_third
from geometry.point2_orientation import point2_orientation_rotate,
    point2_orientation_swap_first_second_neg,
    point2_orientation_swap_first_third_neg,
    point2_orientation_swap_second_third_neg,
    point2_collinear_swap_first_second,
    point2_collinear_swap_first_third,
    point2_collinear_swap_second_third
from geometry.point2_affine import point2_orientation_translate,
    point2_param_line_collinear

attributes Point2[T: OrderedField] {
    /// The signed doubled area of an oriented triangle.
    define triangle_area2(self, b: Point2[T], c: Point2[T]) -> T {
        self.orientation(b, c)
    }

    /// True when an oriented triangle has positive signed doubled area.
    define triangle_ccw(self, b: Point2[T], c: Point2[T]) -> Bool {
        self.triangle_area2(b, c) > T.0
    }

    /// True when an oriented triangle has negative signed doubled area.
    define triangle_cw(self, b: Point2[T], c: Point2[T]) -> Bool {
        self.triangle_area2(b, c) < T.0
    }
}

/// Triangle doubled area is the orientation determinant.
theorem point2_triangle_area2_eq_orientation[T: OrderedField](a: Point2[T], b: Point2[T], c: Point2[T]) {
    a.triangle_area2(b, c) = a.orientation(b, c)
}

/// Positive triangle area is the left-turn predicate.
theorem point2_triangle_ccw_eq_left_turn[T: OrderedField](a: Point2[T], b: Point2[T], c: Point2[T]) {
    a.triangle_ccw(b, c) = a.left_turn(b, c)
}

/// Negative triangle area is the right-turn predicate.
theorem point2_triangle_cw_eq_right_turn[T: OrderedField](a: Point2[T], b: Point2[T], c: Point2[T]) {
    a.triangle_cw(b, c) = a.right_turn(b, c)
}

/// A repeated first edge endpoint gives zero signed doubled area.
theorem point2_triangle_area2_same_first[T: OrderedField](a: Point2[T], b: Point2[T]) {
    a.triangle_area2(a, b) = T.0
} by {
    point2_orientation_same_first(a, b)
}

/// A repeated first and third point gives zero signed doubled area.
theorem point2_triangle_area2_same_second[T: OrderedField](a: Point2[T], b: Point2[T]) {
    a.triangle_area2(b, a) = T.0
} by {
    point2_orientation_same_second(a, b)
}

/// A repeated second edge endpoint gives zero signed doubled area.
theorem point2_triangle_area2_same_third[T: OrderedField](a: Point2[T], b: Point2[T]) {
    a.triangle_area2(b, b) = T.0
} by {
    point2_orientation_same_third(a, b)
}

/// Signed doubled area is invariant under cyclic permutation.
theorem point2_triangle_area2_rotate[T: OrderedField](a: Point2[T], b: Point2[T], c: Point2[T]) {
    a.triangle_area2(b, c) = b.triangle_area2(c, a)
} by {
    point2_orientation_rotate(a, b, c)
}

/// Swapping the first two vertices negates signed doubled area.
theorem point2_triangle_area2_swap_first_second_neg[T: OrderedField](a: Point2[T], b: Point2[T], c: Point2[T]) {
    a.triangle_area2(b, c) = -b.triangle_area2(a, c)
} by {
    point2_orientation_swap_first_second_neg(a, b, c)
}

/// Swapping the first and third vertices negates signed doubled area.
theorem point2_triangle_area2_swap_first_third_neg[T: OrderedField](a: Point2[T], b: Point2[T], c: Point2[T]) {
    a.triangle_area2(b, c) = -c.triangle_area2(b, a)
} by {
    point2_orientation_swap_first_third_neg(a, b, c)
}

/// Swapping the last two vertices negates signed doubled area.
theorem point2_triangle_area2_swap_second_third_neg[T: OrderedField](a: Point2[T], b: Point2[T], c: Point2[T]) {
    a.triangle_area2(b, c) = -a.triangle_area2(c, b)
} by {
    point2_orientation_swap_second_third_neg(a, b, c)
}

/// Zero signed doubled area is collinearity.
theorem point2_triangle_area2_zero_eq_collinear[T: OrderedField](a: Point2[T], b: Point2[T], c: Point2[T]) {
    (a.triangle_area2(b, c) = T.0) = a.collinear(b, c)
}

/// Collinear triangle vertices have zero signed doubled area.
theorem point2_collinear_imp_triangle_area2_zero[T: OrderedField](a: Point2[T], b: Point2[T], c: Point2[T]) {
    a.collinear(b, c) implies a.triangle_area2(b, c) = T.0
}

/// Zero signed doubled area implies collinearity.
theorem point2_triangle_area2_zero_imp_collinear[T: OrderedField](a: Point2[T], b: Point2[T], c: Point2[T]) {
    a.triangle_area2(b, c) = T.0 implies a.collinear(b, c)
}

/// Collinearity is invariant under cyclic permutation as a zero-area fact.
theorem point2_triangle_area2_zero_rotate[T: OrderedField](a: Point2[T], b: Point2[T], c: Point2[T]) {
    (a.triangle_area2(b, c) = T.0) = (b.triangle_area2(c, a) = T.0)
} by {
    point2_triangle_area2_rotate(a, b, c)
}

/// Zero area is invariant under swapping the first two vertices.
theorem point2_triangle_area2_zero_swap_first_second[T: OrderedField](a: Point2[T], b: Point2[T], c: Point2[T]) {
    (a.triangle_area2(b, c) = T.0) = (b.triangle_area2(a, c) = T.0)
} by {
    point2_collinear_swap_first_second(a, b, c)
}

/// Zero area is invariant under swapping the first and third vertices.
theorem point2_triangle_area2_zero_swap_first_third[T: OrderedField](a: Point2[T], b: Point2[T], c: Point2[T]) {
    (a.triangle_area2(b, c) = T.0) = (c.triangle_area2(b, a) = T.0)
} by {
    point2_collinear_swap_first_third(a, b, c)
}

/// Zero area is invariant under swapping the last two vertices.
theorem point2_triangle_area2_zero_swap_second_third[T: OrderedField](a: Point2[T], b: Point2[T], c: Point2[T]) {
    (a.triangle_area2(b, c) = T.0) = (a.triangle_area2(c, b) = T.0)
} by {
    point2_collinear_swap_second_third(a, b, c)
}

/// Translating all vertices preserves signed doubled area.
theorem point2_triangle_area2_translate[T: OrderedField](a: Point2[T], b: Point2[T], c: Point2[T], v: Point2[T]) {
    a.translate(v).triangle_area2(b.translate(v), c.translate(v)) = a.triangle_area2(b, c)
} by {
    point2_orientation_translate(a, b, c, v)
}

/// Translating all vertices preserves positive orientation of triangle area.
theorem point2_triangle_ccw_translate[T: OrderedField](a: Point2[T], b: Point2[T], c: Point2[T], v: Point2[T]) {
    a.translate(v).triangle_ccw(b.translate(v), c.translate(v)) = a.triangle_ccw(b, c)
} by {
    point2_triangle_area2_translate(a, b, c, v)
}

/// Translating all vertices preserves negative orientation of triangle area.
theorem point2_triangle_cw_translate[T: OrderedField](a: Point2[T], b: Point2[T], c: Point2[T], v: Point2[T]) {
    a.translate(v).triangle_cw(b.translate(v), c.translate(v)) = a.triangle_cw(b, c)
} by {
    point2_triangle_area2_translate(a, b, c, v)
}

/// Positive signed area is invariant under cyclic permutation.
theorem point2_triangle_ccw_rotate[T: OrderedField](a: Point2[T], b: Point2[T], c: Point2[T]) {
    a.triangle_ccw(b, c) = b.triangle_ccw(c, a)
} by {
    point2_triangle_area2_rotate(a, b, c)
}

/// Negative signed area is invariant under cyclic permutation.
theorem point2_triangle_cw_rotate[T: OrderedField](a: Point2[T], b: Point2[T], c: Point2[T]) {
    a.triangle_cw(b, c) = b.triangle_cw(c, a)
} by {
    point2_triangle_area2_rotate(a, b, c)
}

/// Zero signed area is invariant under translating all vertices.
theorem point2_triangle_area2_zero_translate[T: OrderedField](a: Point2[T], b: Point2[T], c: Point2[T], v: Point2[T]) {
    (a.translate(v).triangle_area2(b.translate(v), c.translate(v)) = T.0) =
    (a.triangle_area2(b, c) = T.0)
} by {
    point2_triangle_area2_translate(a, b, c, v)
}

/// A parameter-line point on an edge gives zero area with the edge endpoints.
theorem point2_triangle_area2_param_line_zero[T: OrderedField](a: Point2[T], b: Point2[T], t: T) {
    a.triangle_area2(b, a.param_line(b, t)) = T.0
} by {
    point2_param_line_collinear(a, b, t)
}

/// A parameter-line point gives zero area when placed in the middle vertex.
theorem point2_triangle_area2_param_line_zero_middle[T: OrderedField](a: Point2[T], b: Point2[T], t: T) {
    a.triangle_area2(a.param_line(b, t), b) = T.0
} by {
    point2_triangle_area2_param_line_zero(a, b, t)
    point2_triangle_area2_zero_swap_second_third(a, b, a.param_line(b, t))
}

/// A parameter-line point gives zero area when placed in the first vertex.
theorem point2_triangle_area2_param_line_zero_first[T: OrderedField](a: Point2[T], b: Point2[T], t: T) {
    a.param_line(b, t).triangle_area2(a, b) = T.0
} by {
    point2_triangle_area2_param_line_zero(a, b, t)
    point2_triangle_area2_zero_swap_first_third(a, b, a.param_line(b, t))
}

/// A parameter-line point on an edge is collinear with the edge endpoints.
theorem point2_triangle_param_line_collinear[T: OrderedField](a: Point2[T], b: Point2[T], t: T) {
    a.collinear(b, a.param_line(b, t))
} by {
    point2_param_line_collinear(a, b, t)
}
