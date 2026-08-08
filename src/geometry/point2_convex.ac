from ordered_field import OrderedField
from algebra.add_ordered_group import neg_le_neg, le_of_neg_le_neg
from list import List
from geometry.point2 import Point2
from geometry.point2_affine import point2_orientation_translate
from geometry.point2_orientation import point2_collinear_same_first,
    point2_collinear_same_second, point2_collinear_same_third,
    point2_orientation_swap_first_second_neg
from geometry.point2_turning import point2_all_left_turns, point2_all_right_turns

attributes Point2[T: OrderedField] {
    /// True when three points form a nonnegative oriented turn.
    define weak_left_turn(self, b: Point2[T], c: Point2[T]) -> Bool {
        self.orientation(b, c) >= T.0
    }

    /// True when three points form a nonpositive oriented turn.
    define weak_right_turn(self, b: Point2[T], c: Point2[T]) -> Bool {
        self.orientation(b, c) <= T.0
    }
}

/// True when every consecutive triple in a chain is a weak left turn.
define point2_weak_left_turn_chain[T: OrderedField](first: Point2[T], second: Point2[T], rest: List[Point2[T]]) -> Bool {
    match rest {
        List.nil {
            true
        }
        List.cons(third, tail) {
            if first.weak_left_turn(second, third) {
                point2_weak_left_turn_chain(second, third, tail)
            } else {
                false
            }
        }
    }
}

/// True when every consecutive triple in a chain is a weak right turn.
define point2_weak_right_turn_chain[T: OrderedField](first: Point2[T], second: Point2[T], rest: List[Point2[T]]) -> Bool {
    match rest {
        List.nil {
            true
        }
        List.cons(third, tail) {
            if first.weak_right_turn(second, third) {
                point2_weak_right_turn_chain(second, third, tail)
            } else {
                false
            }
        }
    }
}

/// True when every consecutive triple in a point list is a weak left turn.
define point2_all_weak_left_turns[T: OrderedField](points: List[Point2[T]]) -> Bool {
    match points {
        List.nil {
            true
        }
        List.cons(first, tail) {
            match tail {
                List.nil {
                    true
                }
                List.cons(second, rest) {
                    point2_weak_left_turn_chain(first, second, rest)
                }
            }
        }
    }
}

/// True when every consecutive triple in a point list is a weak right turn.
define point2_all_weak_right_turns[T: OrderedField](points: List[Point2[T]]) -> Bool {
    match points {
        List.nil {
            true
        }
        List.cons(first, tail) {
            match tail {
                List.nil {
                    true
                }
                List.cons(second, rest) {
                    point2_weak_right_turn_chain(first, second, rest)
                }
            }
        }
    }
}

/// True when a point list is strictly convex with counterclockwise consecutive turns.
define point2_strictly_convex_ccw[T: OrderedField](points: List[Point2[T]]) -> Bool {
    point2_all_left_turns(points)
}

/// True when a point list is strictly convex with clockwise consecutive turns.
define point2_strictly_convex_cw[T: OrderedField](points: List[Point2[T]]) -> Bool {
    point2_all_right_turns(points)
}

/// True when a point list is weakly convex with counterclockwise consecutive turns.
define point2_weakly_convex_ccw[T: OrderedField](points: List[Point2[T]]) -> Bool {
    point2_all_weak_left_turns(points)
}

/// True when a point list is weakly convex with clockwise consecutive turns.
define point2_weakly_convex_cw[T: OrderedField](points: List[Point2[T]]) -> Bool {
    point2_all_weak_right_turns(points)
}

/// Weak left turns are nonnegative orientations.
theorem point2_weak_left_turn_eq_orientation_nonneg[T: OrderedField](a: Point2[T], b: Point2[T], c: Point2[T]) {
    a.weak_left_turn(b, c) = (a.orientation(b, c) >= T.0)
}

/// Weak right turns are nonpositive orientations.
theorem point2_weak_right_turn_eq_orientation_nonpos[T: OrderedField](a: Point2[T], b: Point2[T], c: Point2[T]) {
    a.weak_right_turn(b, c) = (a.orientation(b, c) <= T.0)
}

/// A strict left turn is a weak left turn.
theorem point2_left_turn_imp_weak_left_turn[T: OrderedField](a: Point2[T], b: Point2[T], c: Point2[T]) {
    a.left_turn(b, c) implies a.weak_left_turn(b, c)
} by {
    if a.left_turn(b, c) {
        a.orientation(b, c) >= T.0
    }
}

/// A strict right turn is a weak right turn.
theorem point2_right_turn_imp_weak_right_turn[T: OrderedField](a: Point2[T], b: Point2[T], c: Point2[T]) {
    a.right_turn(b, c) implies a.weak_right_turn(b, c)
} by {
    if a.right_turn(b, c) {
        a.orientation(b, c) <= T.0
    }
}

/// Collinear points form a weak left turn.
theorem point2_collinear_imp_weak_left_turn[T: OrderedField](a: Point2[T], b: Point2[T], c: Point2[T]) {
    a.collinear(b, c) implies a.weak_left_turn(b, c)
} by {
    if a.collinear(b, c) {
        a.orientation(b, c) >= T.0
    }
}

/// Collinear points form a weak right turn.
theorem point2_collinear_imp_weak_right_turn[T: OrderedField](a: Point2[T], b: Point2[T], c: Point2[T]) {
    a.collinear(b, c) implies a.weak_right_turn(b, c)
} by {
    if a.collinear(b, c) {
        a.orientation(b, c) <= T.0
    }
}

/// A repeated first edge forms a weak left turn.
theorem point2_weak_left_turn_same_first[T: OrderedField](a: Point2[T], b: Point2[T]) {
    a.weak_left_turn(a, b)
} by {
    point2_collinear_same_first(a, b)
    point2_collinear_imp_weak_left_turn(a, a, b)
}

/// A repeated first and third point forms a weak left turn.
theorem point2_weak_left_turn_same_second[T: OrderedField](a: Point2[T], b: Point2[T]) {
    a.weak_left_turn(b, a)
} by {
    point2_collinear_same_second(a, b)
    point2_collinear_imp_weak_left_turn(a, b, a)
}

/// A repeated second edge forms a weak left turn.
theorem point2_weak_left_turn_same_third[T: OrderedField](a: Point2[T], b: Point2[T]) {
    a.weak_left_turn(b, b)
} by {
    point2_collinear_same_third(a, b)
    point2_collinear_imp_weak_left_turn(a, b, b)
}

/// A repeated first edge forms a weak right turn.
theorem point2_weak_right_turn_same_first[T: OrderedField](a: Point2[T], b: Point2[T]) {
    a.weak_right_turn(a, b)
} by {
    point2_collinear_same_first(a, b)
    point2_collinear_imp_weak_right_turn(a, a, b)
}

/// A repeated first and third point forms a weak right turn.
theorem point2_weak_right_turn_same_second[T: OrderedField](a: Point2[T], b: Point2[T]) {
    a.weak_right_turn(b, a)
} by {
    point2_collinear_same_second(a, b)
    point2_collinear_imp_weak_right_turn(a, b, a)
}

/// A repeated second edge forms a weak right turn.
theorem point2_weak_right_turn_same_third[T: OrderedField](a: Point2[T], b: Point2[T]) {
    a.weak_right_turn(b, b)
} by {
    point2_collinear_same_third(a, b)
    point2_collinear_imp_weak_right_turn(a, b, b)
}

/// Reversing the first edge turns weak left turns into weak right turns.
theorem point2_weak_left_turn_reverse_first_edge[T: OrderedField](a: Point2[T], b: Point2[T], c: Point2[T]) {
    b.weak_left_turn(a, c) = a.weak_right_turn(b, c)
} by {
    point2_orientation_swap_first_second_neg(b, a, c)
    if b.weak_left_turn(a, c) {
        b.orientation(a, c) >= T.0
        b.orientation(a, c) = -a.orientation(b, c)
        -a.orientation(b, c) >= T.0
        -a.orientation(b, c) >= -T.0
        le_of_neg_le_neg(a.orientation(b, c), T.0)
        a.weak_right_turn(b, c)
    }
    if a.weak_right_turn(b, c) {
        neg_le_neg(a.orientation(b, c), T.0)
        T.0 <= -a.orientation(b, c)
        b.weak_left_turn(a, c)
    }
}

/// Reversing the first edge turns weak right turns into weak left turns.
theorem point2_weak_right_turn_reverse_first_edge[T: OrderedField](a: Point2[T], b: Point2[T], c: Point2[T]) {
    b.weak_right_turn(a, c) = a.weak_left_turn(b, c)
} by {
    point2_orientation_swap_first_second_neg(b, a, c)
    if b.weak_right_turn(a, c) {
        b.orientation(a, c) <= T.0
        b.orientation(a, c) = -a.orientation(b, c)
        -a.orientation(b, c) <= T.0
        -a.orientation(b, c) <= -T.0
        le_of_neg_le_neg(T.0, a.orientation(b, c))
        a.weak_left_turn(b, c)
    }
    if a.weak_left_turn(b, c) {
        neg_le_neg(T.0, a.orientation(b, c))
        -T.0 = T.0
        -a.orientation(b, c) <= T.0
        b.weak_right_turn(a, c)
    }
}

/// Translating all three points preserves weak left turns.
theorem point2_weak_left_turn_translate[T: OrderedField](a: Point2[T], b: Point2[T], c: Point2[T], v: Point2[T]) {
    a.translate(v).weak_left_turn(b.translate(v), c.translate(v)) = a.weak_left_turn(b, c)
} by {
    point2_orientation_translate(a, b, c, v)
}

/// Translating all three points preserves weak right turns.
theorem point2_weak_right_turn_translate[T: OrderedField](a: Point2[T], b: Point2[T], c: Point2[T], v: Point2[T]) {
    a.translate(v).weak_right_turn(b.translate(v), c.translate(v)) = a.weak_right_turn(b, c)
} by {
    point2_orientation_translate(a, b, c, v)
}

/// An empty weak-left-turn chain is vacuously true.
theorem point2_weak_left_turn_chain_nil[T: OrderedField](first: Point2[T], second: Point2[T]) {
    point2_weak_left_turn_chain(first, second, List.nil[Point2[T]])
}

/// A one-triple weak-left-turn chain is the weak-left-turn predicate of that triple.
theorem point2_weak_left_turn_chain_single[T: OrderedField](first: Point2[T], second: Point2[T], third: Point2[T]) {
    point2_weak_left_turn_chain(first, second, List.cons(third, List.nil[Point2[T]])) = first.weak_left_turn(second, third)
} by {
    if first.weak_left_turn(second, third) {
        point2_weak_left_turn_chain(first, second, List.cons(third, List.nil[Point2[T]]))
    } else {
        not point2_weak_left_turn_chain(first, second, List.cons(third, List.nil[Point2[T]]))
    }
}

/// An empty weak-right-turn chain is vacuously true.
theorem point2_weak_right_turn_chain_nil[T: OrderedField](first: Point2[T], second: Point2[T]) {
    point2_weak_right_turn_chain(first, second, List.nil[Point2[T]])
}

/// A one-triple weak-right-turn chain is the weak-right-turn predicate of that triple.
theorem point2_weak_right_turn_chain_single[T: OrderedField](first: Point2[T], second: Point2[T], third: Point2[T]) {
    point2_weak_right_turn_chain(first, second, List.cons(third, List.nil[Point2[T]])) = first.weak_right_turn(second, third)
} by {
    if first.weak_right_turn(second, third) {
        point2_weak_right_turn_chain(first, second, List.cons(third, List.nil[Point2[T]]))
    } else {
        not point2_weak_right_turn_chain(first, second, List.cons(third, List.nil[Point2[T]]))
    }
}

/// Empty point lists have all consecutive triples weak-left-turning.
theorem point2_all_weak_left_turns_nil[T: OrderedField] {
    point2_all_weak_left_turns(List.nil[Point2[T]])
}

/// Singleton point lists have all consecutive triples weak-left-turning.
theorem point2_all_weak_left_turns_single[T: OrderedField](a: Point2[T]) {
    point2_all_weak_left_turns(List.cons(a, List.nil[Point2[T]]))
}

/// Two-point lists have all consecutive triples weak-left-turning.
theorem point2_all_weak_left_turns_pair[T: OrderedField](a: Point2[T], b: Point2[T]) {
    point2_all_weak_left_turns(List.cons(a, List.cons(b, List.nil[Point2[T]])))
}

/// A three-point list is weak-left-turning exactly when its only triple is a weak left turn.
theorem point2_all_weak_left_turns_triple[T: OrderedField](a: Point2[T], b: Point2[T], c: Point2[T]) {
    point2_all_weak_left_turns(List.cons(a, List.cons(b, List.cons(c, List.nil[Point2[T]])))) = a.weak_left_turn(b, c)
} by {
    point2_weak_left_turn_chain_single(a, b, c)
}

/// Empty point lists have all consecutive triples weak-right-turning.
theorem point2_all_weak_right_turns_nil[T: OrderedField] {
    point2_all_weak_right_turns(List.nil[Point2[T]])
}

/// Singleton point lists have all consecutive triples weak-right-turning.
theorem point2_all_weak_right_turns_single[T: OrderedField](a: Point2[T]) {
    point2_all_weak_right_turns(List.cons(a, List.nil[Point2[T]]))
}

/// Two-point lists have all consecutive triples weak-right-turning.
theorem point2_all_weak_right_turns_pair[T: OrderedField](a: Point2[T], b: Point2[T]) {
    point2_all_weak_right_turns(List.cons(a, List.cons(b, List.nil[Point2[T]])))
}

/// A three-point list is weak-right-turning exactly when its only triple is a weak right turn.
theorem point2_all_weak_right_turns_triple[T: OrderedField](a: Point2[T], b: Point2[T], c: Point2[T]) {
    point2_all_weak_right_turns(List.cons(a, List.cons(b, List.cons(c, List.nil[Point2[T]])))) = a.weak_right_turn(b, c)
} by {
    point2_weak_right_turn_chain_single(a, b, c)
}

/// Strict counterclockwise convexity is all-left-turns by definition.
theorem point2_strictly_convex_ccw_eq_all_left_turns[T: OrderedField](points: List[Point2[T]]) {
    point2_strictly_convex_ccw(points) = point2_all_left_turns(points)
}

/// Strict clockwise convexity is all-right-turns by definition.
theorem point2_strictly_convex_cw_eq_all_right_turns[T: OrderedField](points: List[Point2[T]]) {
    point2_strictly_convex_cw(points) = point2_all_right_turns(points)
}

/// Weak counterclockwise convexity is all weak-left-turns by definition.
theorem point2_weakly_convex_ccw_eq_all_weak_left_turns[T: OrderedField](points: List[Point2[T]]) {
    point2_weakly_convex_ccw(points) = point2_all_weak_left_turns(points)
}

/// Weak clockwise convexity is all weak-right-turns by definition.
theorem point2_weakly_convex_cw_eq_all_weak_right_turns[T: OrderedField](points: List[Point2[T]]) {
    point2_weakly_convex_cw(points) = point2_all_weak_right_turns(points)
}
