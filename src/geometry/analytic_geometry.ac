/// Analytic geometry: the distance formula, the midpoint formula, the
/// point-slope equation of a line, the slope between two points, and the
/// equation of a circle in the coordinate plane, with concrete coordinate
/// verifications.
///
/// Status:
///   - The distance formula states that the distance between (x1, y1) and
///     (x2, y2) is ((x2-x1)^2 + (y2-y1)^2).sqrt, and the 3-4-5 instance — the
///     distance between (0, 0) and (3, 4) is 5 — is verified.
///   - The midpoint formula states that the midpoint of (x1, y1) and (x2, y2)
///     is ((x1+x2)/2, (y1+y2)/2); the midpoint of (0, 0) and (2, 4) is
///     verified to be (1, 2), and the midpoint is equidistant from the two
///     endpoints.
///   - The point-slope equation of a line: the line through (x1, y1) with
///     slope m is y - y1 = m (x - x1); the line through the origin with slope
///     two is verified to be the graph y = 2x and to contain (1, 2).
///   - The slope between two points is m = (y2-y1)/(x2-x1); the slope between
///     (1, 2) and (3, 6) is verified to be two, and a point with nonzero run
///     lies on its own slope line.
///   - The equation of a circle: (x-a)^2 + (y-b)^2 = r^2 holds exactly on the
///     circle with center (a, b) and squared radius r^2; the unit circle
///     centered at the origin of radius one is verified to contain (1, 0)
///     and (0, 1).

from real import Real, sqrt_mul_self, sqrt_value_nonneg, square_le_square_of_nonneg, mul_div_cancel,
    pos_gt_zero, lt_add_pos, lt_trans
from algebra.ring.ring import mul_neg_neg
from order import lt_imp_lte, lte_antisymm
from geometry.point2 import Point2, point2_ext
from geometry.point2_algebra import point2_scalar_sub_reverse_neg
from geometry.point2_metric import point2_dist_sq_nonneg
from geometry.point2_incidence_extra import point2_dist_sq_coordinate_formula
from geometry.point2_circle import point2_on_circle_eq_dist_sq
from geometry.triangle_geometry_arith import real_two, real_three, real_four,
    real_five, real_six, real_twentyfive, real_two_div_two, real_four_div_two,
    real_five_sq_is_twentyfive, real_two_ne_zero, div_add_distrib_local

numerals Real

// ---------------------------------------------------------------------------
// Real arithmetic for the concrete coordinate examples.
// ---------------------------------------------------------------------------

/// Five is nonnegative.
theorem real_five_nonneg {
    real_five >= Real.0
} by {
    Real.1.is_positive
    pos_gt_zero(Real.1)
    Real.0 < Real.1
    Real.1.is_positive
    lt_add_pos(Real.1, Real.1)
    Real.1 < Real.1 + Real.1
    Real.1 < real_two
    lt_trans(Real.0, Real.1, real_two)
    Real.0 < real_two
    Real.1.is_positive
    lt_add_pos(real_two, Real.1)
    real_two < real_two + Real.1
    real_two < real_three
    lt_trans(Real.0, real_two, real_three)
    Real.0 < real_three
    Real.1.is_positive
    lt_add_pos(real_three, Real.1)
    real_three < real_three + Real.1
    real_three < real_four
    lt_trans(Real.0, real_three, real_four)
    Real.0 < real_four
    Real.1.is_positive
    lt_add_pos(real_four, Real.1)
    real_four < real_four + Real.1
    real_four < real_five
    lt_trans(Real.0, real_four, real_five)
    Real.0 < real_five
    lt_imp_lte(Real.0, real_five)
    Real.0 <= real_five
}

/// Six minus two is four.
theorem real_six_sub_two_is_four {
    real_six - real_two = real_four
} by {
    real_six = real_four + real_two
    real_six - real_two = (real_four + real_two) - real_two
    (real_four + real_two) - real_two = real_four
}

/// Three minus one is two.
theorem real_three_sub_one_is_two {
    real_three - Real.1 = real_two
} by { }

// ---------------------------------------------------------------------------
// The distance formula.
// ---------------------------------------------------------------------------

/// The Euclidean distance between two points of the coordinate plane: the
/// nonnegative square root of the squared coordinate distance.
define point_dist(a: Point2[Real], b: Point2[Real]) -> Option[Real] {
    (a.dist_sq(b)).sqrt
}

/// A nonnegative square root is unique: any nonnegative `y` whose square is
/// `x` is the square root of `x`.
theorem real_sqrt_unique(x: Real, y: Real) {
    x >= Real.0 and y >= Real.0 and y * y = x implies x.sqrt = Option.some(y)
} by {
    if x >= Real.0 and y >= Real.0 and y * y = x {
        sqrt_mul_self(x)
        exists(z: Real) {
            x.sqrt = Option.some(z) and z * z = x
        }
        let z: Real satisfy {
            x.sqrt = Option.some(z) and z * z = x
        }
        sqrt_value_nonneg(x, z)
        z >= Real.0
        z * z = x
        y * y = x
        z * z = y * y
        square_le_square_of_nonneg(z, y)
        z <= y
        square_le_square_of_nonneg(y, z)
        y <= z
        lte_antisymm(z, y)
        z = y
        y = z
        x.sqrt = Option.some(z)
        x.sqrt = Option.some(y)
    }
}

/// The Euclidean distance between (x1, y1) and (x2, y2) is the square root of
/// (x2-x1)^2 + (y2-y1)^2.
theorem point_dist_coordinates(x1: Real, y1: Real, x2: Real, y2: Real) {
    point_dist(Point2.new(x2, y2), Point2.new(x1, y1)) =
        ((x2 - x1) * (x2 - x1) + (y2 - y1) * (y2 - y1)).sqrt
} by {
    point2_dist_sq_coordinate_formula(Point2.new(x2, y2), Point2.new(x1, y1))
    Point2.new(x2, y2).dist_sq(Point2.new(x1, y1)) =
        (x2 - x1) * (x2 - x1) + (y2 - y1) * (y2 - y1)
    point_dist(Point2.new(x2, y2), Point2.new(x1, y1)) =
        ((x2 - x1) * (x2 - x1) + (y2 - y1) * (y2 - y1)).sqrt
}

/// The distance formula: any nonnegative `d` whose square is the radicand
/// (x2-x1)^2 + (y2-y1)^2 is the distance between (x1, y1) and (x2, y2).
theorem distance_formula(x1: Real, y1: Real, x2: Real, y2: Real, d: Real) {
    d >= Real.0 and d * d = (x2 - x1) * (x2 - x1) + (y2 - y1) * (y2 - y1) implies
    point_dist(Point2.new(x2, y2), Point2.new(x1, y1)) = Option.some(d)
} by {
    if d >= Real.0 and d * d = (x2 - x1) * (x2 - x1) + (y2 - y1) * (y2 - y1) {
        point_dist_coordinates(x1, y1, x2, y2)
        point_dist(Point2.new(x2, y2), Point2.new(x1, y1)) =
            ((x2 - x1) * (x2 - x1) + (y2 - y1) * (y2 - y1)).sqrt
        point2_dist_sq_coordinate_formula(Point2.new(x2, y2), Point2.new(x1, y1))
        Point2.new(x2, y2).dist_sq(Point2.new(x1, y1)) =
            (x2 - x1) * (x2 - x1) + (y2 - y1) * (y2 - y1)
        point2_dist_sq_nonneg(Point2.new(x2, y2), Point2.new(x1, y1))
        Point2.new(x2, y2).dist_sq(Point2.new(x1, y1)) >= Real.0
        (x2 - x1) * (x2 - x1) + (y2 - y1) * (y2 - y1) >= Real.0
        real_sqrt_unique((x2 - x1) * (x2 - x1) + (y2 - y1) * (y2 - y1), d)
        ((x2 - x1) * (x2 - x1) + (y2 - y1) * (y2 - y1)).sqrt = Option.some(d)
        point_dist(Point2.new(x2, y2), Point2.new(x1, y1)) = Option.some(d)
    }
}

/// The distance between (0, 0) and (3, 4) is five: the 3-4-5 instance of the
/// distance formula.
theorem distance_three_four_five {
    point_dist(Point2.new(real_three, real_four), Point2.new(Real.0, Real.0)) =
        Option.some(real_five)
} by {
    distance_formula(Real.0, Real.0, real_three, real_four, real_five)
    real_five_nonneg
    real_five >= Real.0
    (real_three - Real.0) * (real_three - Real.0) +
        (real_four - Real.0) * (real_four - Real.0) = real_twentyfive
    real_five_sq_is_twentyfive
    real_five * real_five = real_twentyfive
    real_five * real_five =
        (real_three - Real.0) * (real_three - Real.0) +
        (real_four - Real.0) * (real_four - Real.0)
    real_five >= Real.0 and real_five * real_five =
        (real_three - Real.0) * (real_three - Real.0) +
        (real_four - Real.0) * (real_four - Real.0)
    point_dist(Point2.new(real_three, real_four), Point2.new(Real.0, Real.0)) =
        Option.some(real_five)
}

// ---------------------------------------------------------------------------
// The midpoint formula.
// ---------------------------------------------------------------------------

/// The midpoint of two points: the coordinatewise average of their
/// coordinates.
define point_midpoint(a: Point2[Real], b: Point2[Real]) -> Point2[Real] {
    Point2.new((a.x + b.x) / real_two, (a.y + b.y) / real_two)
}

/// The x-coordinate of a midpoint is the average of the x-coordinates.
theorem point_midpoint_x(a: Point2[Real], b: Point2[Real]) {
    point_midpoint(a, b).x = (a.x + b.x) / real_two
} by { }

/// The y-coordinate of a midpoint is the average of the y-coordinates.
theorem point_midpoint_y(a: Point2[Real], b: Point2[Real]) {
    point_midpoint(a, b).y = (a.y + b.y) / real_two
} by { }

/// A number is twice its half: x/2 + x/2 = x.
theorem real_half_twice(a: Real) {
    a / real_two + a / real_two = a
} by {
    mul_div_cancel(a, real_two)
    real_two_ne_zero
    real_two * (a / real_two) = a
    real_two * (a / real_two) = (Real.1 + Real.1) * (a / real_two)
    (Real.1 + Real.1) * (a / real_two) = a / real_two + a / real_two
    a / real_two + a / real_two = a
}

/// Shifting an endpoint to the midpoint: x - (x + y)/2 = (x - y)/2.
theorem real_average_shift(a: Real, b: Real) {
    a - (a + b) / real_two = (a - b) / real_two
} by {
    div_add_distrib_local(a, b, real_two)
    real_two_ne_zero
    (a + b) / real_two = a / real_two + b / real_two
    a - ((a + b) / real_two) = a - (a / real_two + b / real_two)
    a - (a / real_two + b / real_two) = (a - a / real_two) - b / real_two
    real_half_twice(a)
    a / real_two + a / real_two = a
    a - a / real_two = a / real_two
    (a - a / real_two) - b / real_two = a / real_two - b / real_two
    div_add_distrib_local(a, -b, real_two)
    real_two_ne_zero
    (a - b) / real_two = a / real_two + (-b) / real_two
    a / real_two - b / real_two = (a - b) / real_two
    a - (a + b) / real_two = (a - b) / real_two
}

/// The square of half a difference is unchanged when the difference is
/// reversed.
theorem real_div_sq_reverse(a: Real, b: Real) {
    ((a - b) / real_two) * ((a - b) / real_two) =
        ((b - a) / real_two) * ((b - a) / real_two)
} by {
    point2_scalar_sub_reverse_neg(a, b)
    a - b = -(b - a)
    (a - b) / real_two = (-(b - a)) / real_two
    (-(b - a)) / real_two = -((b - a) / real_two)
    (a - b) / real_two = -((b - a) / real_two)
    ((a - b) / real_two) * ((a - b) / real_two) =
        (-((b - a) / real_two)) * (-((b - a) / real_two))
    mul_neg_neg[Real]((b - a) / real_two, (b - a) / real_two)
    (-((b - a) / real_two)) * (-((b - a) / real_two)) =
        ((b - a) / real_two) * ((b - a) / real_two)
    ((a - b) / real_two) * ((a - b) / real_two) =
        ((b - a) / real_two) * ((b - a) / real_two)
}

/// The midpoint of two points is equidistant from them.
theorem point_midpoint_equidistant(a: Point2[Real], b: Point2[Real]) {
    a.dist_sq(point_midpoint(a, b)) = b.dist_sq(point_midpoint(a, b))
} by {
    point2_dist_sq_coordinate_formula(a, point_midpoint(a, b))
    point2_dist_sq_coordinate_formula(b, point_midpoint(a, b))
    real_average_shift(a.x, b.x)
    a.x - point_midpoint(a, b).x = (a.x - b.x) / real_two
    real_average_shift(b.x, a.x)
    b.x - point_midpoint(a, b).x = (b.x - a.x) / real_two
    real_average_shift(a.y, b.y)
    a.y - point_midpoint(a, b).y = (a.y - b.y) / real_two
    real_average_shift(b.y, a.y)
    b.y - point_midpoint(a, b).y = (b.y - a.y) / real_two
    real_div_sq_reverse(a.x, b.x)
    real_div_sq_reverse(a.y, b.y)
    a.dist_sq(point_midpoint(a, b)) =
        ((a.x - b.x) / real_two) * ((a.x - b.x) / real_two) +
        ((a.y - b.y) / real_two) * ((a.y - b.y) / real_two)
    b.dist_sq(point_midpoint(a, b)) =
        ((b.x - a.x) / real_two) * ((b.x - a.x) / real_two) +
        ((b.y - a.y) / real_two) * ((b.y - a.y) / real_two)
    a.dist_sq(point_midpoint(a, b)) =
        ((b.x - a.x) / real_two) * ((b.x - a.x) / real_two) +
        ((b.y - a.y) / real_two) * ((b.y - a.y) / real_two)
    a.dist_sq(point_midpoint(a, b)) = b.dist_sq(point_midpoint(a, b))
}

/// The midpoint of (0, 0) and (2, 4) is (1, 2).
theorem midpoint_of_origin_and_two_four {
    point_midpoint(Point2.new(Real.0, Real.0), Point2.new(real_two, real_four)) =
        Point2.new(Real.1, real_two)
} by {
    point_midpoint_x(Point2.new(Real.0, Real.0), Point2.new(real_two, real_four))
    point_midpoint(Point2.new(Real.0, Real.0), Point2.new(real_two, real_four)).x =
        (Real.0 + real_two) / real_two
    (Real.0 + real_two) / real_two = real_two / real_two
    real_two_div_two
    real_two / real_two = Real.1
    point_midpoint(Point2.new(Real.0, Real.0), Point2.new(real_two, real_four)).x = Real.1
    point_midpoint_y(Point2.new(Real.0, Real.0), Point2.new(real_two, real_four))
    point_midpoint(Point2.new(Real.0, Real.0), Point2.new(real_two, real_four)).y =
        (Real.0 + real_four) / real_two
    (Real.0 + real_four) / real_two = real_four / real_two
    real_four_div_two
    real_four / real_two = real_two
    point_midpoint(Point2.new(Real.0, Real.0), Point2.new(real_two, real_four)).y = real_two
    point2_ext(point_midpoint(Point2.new(Real.0, Real.0), Point2.new(real_two, real_four)),
        Point2.new(Real.1, real_two))
}

// ---------------------------------------------------------------------------
// The equation of a line.
// ---------------------------------------------------------------------------

/// True when `p` lies on the line through `anchor` with slope `m`: the
/// point-slope form y - y1 = m (x - x1).
define on_line_slope(anchor: Point2[Real], m: Real, p: Point2[Real]) -> Bool {
    p.y - anchor.y = m * (p.x - anchor.x)
}

/// The point-slope equation of a line: the line through (x1, y1) with slope
/// `m` is y - y1 = m (x - x1).
theorem point_slope_line_equation(x1: Real, y1: Real, m: Real, x: Real, y: Real) {
    on_line_slope(Point2.new(x1, y1), m, Point2.new(x, y)) = (y - y1 = m * (x - x1))
} by {
    on_line_slope(Point2.new(x1, y1), m, Point2.new(x, y)) =
        (Point2.new(x, y).y - Point2.new(x1, y1).y =
         m * (Point2.new(x, y).x - Point2.new(x1, y1).x))
    Point2.new(x, y).y - Point2.new(x1, y1).y = y - y1
    Point2.new(x, y).x - Point2.new(x1, y1).x = x - x1
    on_line_slope(Point2.new(x1, y1), m, Point2.new(x, y)) = (y - y1 = m * (x - x1))
}

/// A point lies on its own slope line.
theorem on_line_slope_anchor(anchor: Point2[Real], m: Real) {
    on_line_slope(anchor, m, anchor)
} by {
    on_line_slope(anchor, m, anchor) = (anchor.y - anchor.y = m * (anchor.x - anchor.x))
    anchor.y - anchor.y = Real.0
    anchor.x - anchor.x = Real.0
    m * (anchor.x - anchor.x) = Real.0
    anchor.y - anchor.y = m * (anchor.x - anchor.x)
    on_line_slope(anchor, m, anchor)
}

/// The line through the origin with slope two is the graph y = 2x.
theorem line_through_origin_slope_two_graph(x: Real, y: Real) {
    on_line_slope(Point2.new(Real.0, Real.0), real_two, Point2.new(x, y)) = (y = real_two * x)
} by { }

/// The point (1, 2) lies on the line through the origin with slope two.
theorem line_through_origin_slope_two_contains_one_two {
    on_line_slope(Point2.new(Real.0, Real.0), real_two, Point2.new(Real.1, real_two))
} by { }

// ---------------------------------------------------------------------------
// The slope between two points.
// ---------------------------------------------------------------------------

/// The slope of the line through two points with distinct x-coordinates:
/// the rise over the run.
define slope_between(p: Point2[Real], q: Point2[Real]) -> Real {
    (q.y - p.y) / (q.x - p.x)
}

/// The slope between (x1, y1) and (x2, y2) is (y2 - y1) / (x2 - x1).
theorem slope_between_coordinates(x1: Real, y1: Real, x2: Real, y2: Real) {
    slope_between(Point2.new(x1, y1), Point2.new(x2, y2)) = (y2 - y1) / (x2 - x1)
} by { }

/// When the run is nonzero, the second point lies on the slope line through
/// the first.
theorem slope_between_on_line(p: Point2[Real], q: Point2[Real]) {
    q.x - p.x != Real.0 implies on_line_slope(p, slope_between(p, q), q)
} by {
    if q.x - p.x != Real.0 {
        mul_div_cancel(q.y - p.y, q.x - p.x)
        (q.x - p.x) * ((q.y - p.y) / (q.x - p.x)) = q.y - p.y
        slope_between(p, q) = (q.y - p.y) / (q.x - p.x)
        on_line_slope(p, slope_between(p, q), q)
    }
}

/// The slope between (1, 2) and (3, 6) is two.
theorem slope_between_one_two_and_three_six {
    slope_between(Point2.new(Real.1, real_two), Point2.new(real_three, real_six)) = real_two
} by {
    slope_between_coordinates(Real.1, real_two, real_three, real_six)
    slope_between(Point2.new(Real.1, real_two), Point2.new(real_three, real_six)) =
        (real_six - real_two) / (real_three - Real.1)
    real_six_sub_two_is_four
    real_six - real_two = real_four
    real_three_sub_one_is_two
    real_three - Real.1 = real_two
    (real_six - real_two) / (real_three - Real.1) = real_four / real_two
    real_four_div_two
    real_four / real_two = real_two
    slope_between(Point2.new(Real.1, real_two), Point2.new(real_three, real_six)) = real_two
}

/// The line through (1, 2) with slope two contains (3, 6).
theorem line_slope_two_contains_three_six {
    on_line_slope(Point2.new(Real.1, real_two), real_two, Point2.new(real_three, real_six))
} by { }

// ---------------------------------------------------------------------------
// The equation of a circle.
// ---------------------------------------------------------------------------

/// The equation of a circle: the point (x, y) lies on the circle with center
/// (a, b) and squared radius r^2 exactly when (x-a)^2 + (y-b)^2 = r^2.
theorem circle_equation(a: Real, b: Real, radius_sq: Real, x: Real, y: Real) {
    Point2.new(a, b).on_circle(radius_sq, Point2.new(x, y)) =
        ((x - a) * (x - a) + (y - b) * (y - b) = radius_sq)
} by {
    point2_on_circle_eq_dist_sq(Point2.new(a, b), radius_sq, Point2.new(x, y))
    Point2.new(a, b).on_circle(radius_sq, Point2.new(x, y)) =
        (Point2.new(x, y).dist_sq(Point2.new(a, b)) = radius_sq)
    point2_dist_sq_coordinate_formula(Point2.new(x, y), Point2.new(a, b))
    Point2.new(x, y).dist_sq(Point2.new(a, b)) =
        (x - a) * (x - a) + (y - b) * (y - b)
    Point2.new(a, b).on_circle(radius_sq, Point2.new(x, y)) =
        ((x - a) * (x - a) + (y - b) * (y - b) = radius_sq)
}

/// The unit circle centered at the origin contains (1, 0).
theorem unit_circle_contains_one_zero {
    Point2.new(Real.0, Real.0).on_circle(Real.1, Point2.new(Real.1, Real.0))
} by {
    point2_on_circle_eq_dist_sq(Point2.new(Real.0, Real.0), Real.1, Point2.new(Real.1, Real.0))
    point2_dist_sq_coordinate_formula(Point2.new(Real.1, Real.0), Point2.new(Real.0, Real.0))
    Point2.new(Real.1, Real.0).dist_sq(Point2.new(Real.0, Real.0)) =
        (Real.1 - Real.0) * (Real.1 - Real.0) + (Real.0 - Real.0) * (Real.0 - Real.0)
    (Real.1 - Real.0) * (Real.1 - Real.0) = Real.1
    (Real.0 - Real.0) * (Real.0 - Real.0) = Real.0
    (Real.1 - Real.0) * (Real.1 - Real.0) + (Real.0 - Real.0) * (Real.0 - Real.0) = Real.1
    Point2.new(Real.1, Real.0).dist_sq(Point2.new(Real.0, Real.0)) = Real.1
    Point2.new(Real.0, Real.0).on_circle(Real.1, Point2.new(Real.1, Real.0))
}

/// The unit circle centered at the origin contains (0, 1).
theorem unit_circle_contains_zero_one {
    Point2.new(Real.0, Real.0).on_circle(Real.1, Point2.new(Real.0, Real.1))
} by {
    point2_on_circle_eq_dist_sq(Point2.new(Real.0, Real.0), Real.1, Point2.new(Real.0, Real.1))
    point2_dist_sq_coordinate_formula(Point2.new(Real.0, Real.1), Point2.new(Real.0, Real.0))
    Point2.new(Real.0, Real.1).dist_sq(Point2.new(Real.0, Real.0)) =
        (Real.0 - Real.0) * (Real.0 - Real.0) + (Real.1 - Real.0) * (Real.1 - Real.0)
    (Real.0 - Real.0) * (Real.0 - Real.0) = Real.0
    (Real.1 - Real.0) * (Real.1 - Real.0) = Real.1
    (Real.0 - Real.0) * (Real.0 - Real.0) + (Real.1 - Real.0) * (Real.1 - Real.0) =
        Real.0 + Real.1
    Real.0 + Real.1 = Real.1
    (Real.0 - Real.0) * (Real.0 - Real.0) + (Real.1 - Real.0) * (Real.1 - Real.0) = Real.1
    Point2.new(Real.0, Real.1).dist_sq(Point2.new(Real.0, Real.0)) = Real.1
    Point2.new(Real.0, Real.0).on_circle(Real.1, Point2.new(Real.0, Real.1))
}

/// The unit circle centered at the origin contains both (1, 0) and (0, 1).
theorem unit_circle_contains_one_zero_and_zero_one {
    Point2.new(Real.0, Real.0).same_circle(Real.1,
        Point2.new(Real.1, Real.0), Point2.new(Real.0, Real.1))
} by {
    unit_circle_contains_one_zero
    unit_circle_contains_zero_one
}
