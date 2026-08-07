/// Translation bridge lemmas for Point2 convex point sets and convex hulls.

from ordered_field import OrderedField
from geometry.point2 import Point2
from geometry.point2_affine import point2_param_line_translate, point2_translate_neg_right
from data.basic.set import Set, image_contains, set_ext, set_image, set_image_contains_eq,
    set_image_contains_witness

/// The image of a point set under translation by a fixed vector.
define point2_point_set_translate[T: OrderedField](s: Set[Point2[T]], v: Point2[T]) -> Set[Point2[T]] {
    set_image(s, function(p: Point2[T]) { p.translate(v) })
}

/// A point set is convex when it contains each segment between its points.
define point2_point_set_convex[T: OrderedField](s: Set[Point2[T]]) -> Bool {
    forall(a: Point2[T], b: Point2[T], t: T) {
        s.contains(a) and s.contains(b) and T.0 <= t and t <= T.1 implies
        s.contains(a.param_line(b, t))
    }
}

/// A point lies in the convex hull of `s` when it is in every convex superset of `s`.
define point2_convex_hull_contains[T: OrderedField](s: Set[Point2[T]], p: Point2[T]) -> Bool {
    forall(c: Set[Point2[T]]) {
        point2_point_set_convex(c) and s.subset(c) implies c.contains(p)
    }
}

/// The convex hull of a point set.
define point2_convex_hull[T: OrderedField](s: Set[Point2[T]]) -> Set[Point2[T]] {
    Set[Point2[T]].new(point2_convex_hull_contains(s))
}

/// Membership in a convex hull is the convex-hull predicate.
theorem point2_convex_hull_contains_eq[T: OrderedField](s: Set[Point2[T]], p: Point2[T]) {
    point2_convex_hull(s).contains(p) = point2_convex_hull_contains(s, p)
}

/// Membership in a translated point set has the expected image witness.
theorem point2_point_set_translate_contains_witness[T: OrderedField](s: Set[Point2[T]], v: Point2[T], p: Point2[T]) {
    point2_point_set_translate(s, v).contains(p) implies exists(q: Point2[T]) {
        s.contains(q) and p = q.translate(v)
    }
} by {
    if point2_point_set_translate(s, v).contains(p) {
        point2_point_set_translate(s, v) = set_image(s, function(q: Point2[T]) { q.translate(v) })
        set_image_contains_witness(s, function(q: Point2[T]) { q.translate(v) }, p)
    }
}

/// Translating a member of a point set gives a member of the translated point set.
theorem point2_point_set_translate_contains_forward[T: OrderedField](s: Set[Point2[T]], v: Point2[T], p: Point2[T]) {
    s.contains(p) implies point2_point_set_translate(s, v).contains(p.translate(v))
} by {
    if s.contains(p) {
        exists(q: Point2[T]) {
            q = p and s.contains(q) and p.translate(v) = q.translate(v)
        }
        image_contains(function(q: Point2[T]) { q.translate(v) }, s, p.translate(v))
        point2_point_set_translate(s, v) = set_image(s, function(q: Point2[T]) { q.translate(v) })
        set_image_contains_eq(s, function(q: Point2[T]) { q.translate(v) }, p.translate(v))
        set_image(s, function(q: Point2[T]) { q.translate(v) }).contains(p.translate(v))
        point2_point_set_translate(s, v).contains(p.translate(v))
    }
}

/// Translating a point set by `v` and then by `-v` recovers the original set.
theorem point2_point_set_translate_neg_right[T: OrderedField](s: Set[Point2[T]], v: Point2[T]) {
    point2_point_set_translate(point2_point_set_translate(s, v), v.neg) = s
} by {
    let lhs = point2_point_set_translate(point2_point_set_translate(s, v), v.neg)
    forall(p: Point2[T]) {
        if lhs.contains(p) {
            point2_point_set_translate_contains_witness(point2_point_set_translate(s, v), v.neg, p)
            let q: Point2[T] satisfy {
                point2_point_set_translate(s, v).contains(q) and p = q.translate(v.neg)
            }
            point2_point_set_translate_contains_witness(s, v, q)
            let r: Point2[T] satisfy {
                s.contains(r) and q = r.translate(v)
            }
            p = q.translate(v.neg)
            q = r.translate(v)
            p = r.translate(v).translate(v.neg)
            point2_translate_neg_right(r, v)
            p = r
            s.contains(p)
        }
        if s.contains(p) {
            point2_point_set_translate_contains_forward(s, v, p)
            point2_point_set_translate(s, v).contains(p.translate(v))
            point2_translate_neg_right(p, v)
            p.translate(v).translate(v.neg) = p
            point2_point_set_translate_contains_forward(point2_point_set_translate(s, v), v.neg, p.translate(v))
            lhs.contains(p.translate(v).translate(v.neg))
            lhs.contains(p)
        }
        lhs.contains(p) = s.contains(p)
    }
    set_ext(lhs, s)
}

/// The translated copy of a convex point set is convex.
theorem point2_point_set_translate_convex_forward[T: OrderedField](s: Set[Point2[T]], v: Point2[T]) {
    point2_point_set_convex(s) implies point2_point_set_convex(point2_point_set_translate(s, v))
} by {
    if point2_point_set_convex(s) {
        forall(a: Point2[T], b: Point2[T], t: T) {
            if point2_point_set_translate(s, v).contains(a) and
                point2_point_set_translate(s, v).contains(b) and T.0 <= t and t <= T.1 {
                point2_point_set_translate_contains_witness(s, v, a)
                let a0: Point2[T] satisfy {
                    s.contains(a0) and a = a0.translate(v)
                }
                point2_point_set_translate_contains_witness(s, v, b)
                let b0: Point2[T] satisfy {
                    s.contains(b0) and b = b0.translate(v)
                }
                point2_point_set_convex(s) = forall(x: Point2[T], y: Point2[T], u: T) {
                    s.contains(x) and s.contains(y) and T.0 <= u and u <= T.1 implies
                    s.contains(x.param_line(y, u))
                }
                s.contains(a0.param_line(b0, t))
                point2_param_line_translate(a0, b0, t, v)
                a0.param_line(b0, t).translate(v) = a0.translate(v).param_line(b0.translate(v), t)
                a.param_line(b, t) = a0.param_line(b0, t).translate(v)
                point2_point_set_translate_contains_forward(s, v, a0.param_line(b0, t))
                point2_point_set_translate(s, v).contains(a0.param_line(b0, t).translate(v))
                point2_point_set_translate(s, v).contains(a.param_line(b, t))
            }
        }
    }
}

/// Translation preserves convexity of point sets.
theorem point2_point_set_translate_convex_iff[T: OrderedField](s: Set[Point2[T]], v: Point2[T]) {
    point2_point_set_convex(point2_point_set_translate(s, v)) = point2_point_set_convex(s)
} by {
    if point2_point_set_convex(s) {
        point2_point_set_translate_convex_forward(s, v)
    }
    if point2_point_set_convex(point2_point_set_translate(s, v)) {
        point2_point_set_translate_convex_forward(point2_point_set_translate(s, v), v.neg)
        point2_point_set_translate(point2_point_set_translate(s, v), v.neg) = s
        point2_point_set_convex(s)
    }
}

/// Translating a convex-hull point gives a point in the translated convex hull.
theorem point2_convex_hull_translate_contains_forward[T: OrderedField](s: Set[Point2[T]], v: Point2[T], p: Point2[T]) {
    point2_convex_hull(s).contains(p) implies
    point2_convex_hull(point2_point_set_translate(s, v)).contains(p.translate(v))
} by {
    if point2_convex_hull(s).contains(p) {
        forall(c: Set[Point2[T]]) {
            if point2_point_set_convex(c) and point2_point_set_translate(s, v).subset(c) {
                let d = point2_point_set_translate(c, v.neg)
                point2_point_set_translate_convex_iff(c, v.neg)
                point2_point_set_convex(d)
                forall(x: Point2[T]) {
                    if s.contains(x) {
                        point2_point_set_translate_contains_forward(s, v, x)
                        point2_point_set_translate(s, v).contains(x.translate(v))
                        c.contains(x.translate(v))
                        point2_point_set_translate_contains_forward(c, v.neg, x.translate(v))
                        d.contains(x.translate(v).translate(v.neg))
                        point2_translate_neg_right(x, v)
                        d.contains(x)
                    }
                }
                s.subset(d)
                point2_convex_hull_contains_eq(s, p)
                point2_convex_hull_contains(s, p)
                point2_convex_hull_contains(s, p) = forall(e: Set[Point2[T]]) {
                    point2_point_set_convex(e) and s.subset(e) implies e.contains(p)
                }
                forall(e: Set[Point2[T]]) {
                    point2_point_set_convex(e) and s.subset(e) implies e.contains(p)
                }
                point2_point_set_convex(d) and s.subset(d)
                d.contains(p)
                point2_point_set_translate_contains_witness(c, v.neg, p)
                let q: Point2[T] satisfy {
                    c.contains(q) and p = q.translate(v.neg)
                }
                p.translate(v) = q.translate(v.neg).translate(v)
                point2_translate_neg_right(q, v.neg)
                q.translate(v.neg).translate(v) = q
                p.translate(v) = q
                c.contains(p.translate(v))
            }
        }
        point2_convex_hull_contains(point2_point_set_translate(s, v), p.translate(v))
        point2_convex_hull(point2_point_set_translate(s, v)).contains(p.translate(v))
    }
}

/// Translating a convex-hull point backward gives a point in the original convex hull.
theorem point2_convex_hull_translate_contains_backward[T: OrderedField](s: Set[Point2[T]], v: Point2[T], p: Point2[T]) {
    point2_convex_hull(point2_point_set_translate(s, v)).contains(p.translate(v)) implies
    point2_convex_hull(s).contains(p)
} by {
    if point2_convex_hull(point2_point_set_translate(s, v)).contains(p.translate(v)) {
        point2_convex_hull_translate_contains_forward(point2_point_set_translate(s, v), v.neg, p.translate(v))
        point2_convex_hull(point2_point_set_translate(point2_point_set_translate(s, v), v.neg)).contains(p.translate(v).translate(v.neg))
        point2_point_set_translate(point2_point_set_translate(s, v), v.neg) = s
        point2_translate_neg_right(p, v)
        point2_convex_hull(s).contains(p)
    }
}

/// Translation preserves convex-hull membership exactly.
theorem point2_convex_hull_translate_contains_iff[T: OrderedField](s: Set[Point2[T]], v: Point2[T], p: Point2[T]) {
    point2_convex_hull(point2_point_set_translate(s, v)).contains(p.translate(v)) =
    point2_convex_hull(s).contains(p)
} by {
    if point2_convex_hull(s).contains(p) {
        point2_convex_hull_translate_contains_forward(s, v, p)
    }
    if point2_convex_hull(point2_point_set_translate(s, v)).contains(p.translate(v)) {
        point2_convex_hull_translate_contains_backward(s, v, p)
    }
}
