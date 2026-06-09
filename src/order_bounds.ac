/// Generic least-upper-bound and greatest-lower-bound theory for partial orders.
///
/// These predicates take a membership predicate `p: P -> Bool` standing for a
/// subset of a partial order and characterize its bounds, least and greatest
/// elements, suprema, and infima. They mirror the Mathlib `IsLUB`, `IsGLB`,
/// `IsLeast`, and `IsGreatest` interfaces at the level of a single partial order.

from order import PartialOrder, lte_antisymm
from lattice import MeetSemilattice, JoinSemilattice, Lattice, meet_lte_left, meet_lte_right,
    lte_meet_of_bounds, lte_join_left, lte_join_right, join_lte_of_bounds

/// True if `a` is below every element of `p`.
define is_lower_bound[P: PartialOrder](p: P -> Bool, a: P) -> Bool {
    forall(x: P) {
        p(x) implies a <= x
    }
}

/// True if `b` is above every element of `p`.
define is_upper_bound[P: PartialOrder](p: P -> Bool, b: P) -> Bool {
    forall(x: P) {
        p(x) implies x <= b
    }
}

/// True if `a` is an element of `p` that is below every other element of `p`.
define is_least[P: PartialOrder](p: P -> Bool, a: P) -> Bool {
    p(a) and is_lower_bound(p, a)
}

/// True if `b` is an element of `p` that is above every other element of `p`.
define is_greatest[P: PartialOrder](p: P -> Bool, b: P) -> Bool {
    p(b) and is_upper_bound(p, b)
}

/// True if `a` is the greatest of all lower bounds of `p`, that is, its infimum.
define is_glb[P: PartialOrder](p: P -> Bool, a: P) -> Bool {
    is_lower_bound(p, a) and forall(b: P) {
        is_lower_bound(p, b) implies b <= a
    }
}

/// True if `b` is the least of all upper bounds of `p`, that is, its supremum.
define is_lub[P: PartialOrder](p: P -> Bool, b: P) -> Bool {
    is_upper_bound(p, b) and forall(c: P) {
        is_upper_bound(p, c) implies b <= c
    }
}

// --- Lower and upper bound basics ---

/// A lower bound is below each member of the set.
theorem lower_bound_le[P: PartialOrder](p: P -> Bool, a: P, x: P) {
    is_lower_bound(p, a) and p(x) implies a <= x
} by {
    is_lower_bound(p, a) = forall(y: P) { p(y) implies a <= y }
}

/// An upper bound is above each member of the set.
theorem upper_bound_ge[P: PartialOrder](p: P -> Bool, b: P, x: P) {
    is_upper_bound(p, b) and p(x) implies x <= b
} by {
    is_upper_bound(p, b) = forall(y: P) { p(y) implies y <= b }
}

/// Anything below a lower bound is still a lower bound.
theorem lower_bound_of_le[P: PartialOrder](p: P -> Bool, a: P, c: P) {
    is_lower_bound(p, a) and c <= a implies is_lower_bound(p, c)
} by {
    forall(x: P) {
        if p(x) {
            a <= x
            c <= x
        }
    }
}

/// Anything above an upper bound is still an upper bound.
theorem upper_bound_of_ge[P: PartialOrder](p: P -> Bool, b: P, c: P) {
    is_upper_bound(p, b) and b <= c implies is_upper_bound(p, c)
} by {
    forall(x: P) {
        if p(x) {
            x <= b
            x <= c
        }
    }
}

/// A bound for a larger set bounds any smaller set below.
theorem lower_bound_mono[P: PartialOrder](p: P -> Bool, q: P -> Bool, a: P) {
    is_lower_bound(q, a) and forall(x: P) { p(x) implies q(x) }
        implies is_lower_bound(p, a)
} by {
    forall(x: P) {
        if p(x) {
            q(x)
            a <= x
        }
    }
}

/// A bound for a larger set bounds any smaller set above.
theorem upper_bound_mono[P: PartialOrder](p: P -> Bool, q: P -> Bool, b: P) {
    is_upper_bound(q, b) and forall(x: P) { p(x) implies q(x) }
        implies is_upper_bound(p, b)
} by {
    forall(x: P) {
        if p(x) {
            q(x)
            x <= b
        }
    }
}

// --- Least and greatest elements ---

/// A least element belongs to the set.
theorem is_least_mem[P: PartialOrder](p: P -> Bool, a: P) {
    is_least(p, a) implies p(a)
}

/// A least element is a lower bound.
theorem is_least_is_lower_bound[P: PartialOrder](p: P -> Bool, a: P) {
    is_least(p, a) implies is_lower_bound(p, a)
}

/// A greatest element belongs to the set.
theorem is_greatest_mem[P: PartialOrder](p: P -> Bool, b: P) {
    is_greatest(p, b) implies p(b)
}

/// A greatest element is an upper bound.
theorem is_greatest_is_upper_bound[P: PartialOrder](p: P -> Bool, b: P) {
    is_greatest(p, b) implies is_upper_bound(p, b)
}

/// The least element is below every member of the set.
theorem is_least_le[P: PartialOrder](p: P -> Bool, a: P, x: P) {
    is_least(p, a) and p(x) implies a <= x
}

/// The greatest element is above every member of the set.
theorem is_greatest_ge[P: PartialOrder](p: P -> Bool, b: P, x: P) {
    is_greatest(p, b) and p(x) implies x <= b
}

/// A least element is unique.
theorem is_least_unique[P: PartialOrder](p: P -> Bool, a: P, a2: P) {
    is_least(p, a) and is_least(p, a2) implies a = a2
} by {
    a <= a2
    a2 <= a
    lte_antisymm(a, a2)
}

/// A greatest element is unique.
theorem is_greatest_unique[P: PartialOrder](p: P -> Bool, b: P, b2: P) {
    is_greatest(p, b) and is_greatest(p, b2) implies b = b2
} by {
    b <= b2
    b2 <= b
    lte_antisymm(b, b2)
}

// --- Suprema and infima ---

/// An infimum is a lower bound.
theorem is_glb_is_lower_bound[P: PartialOrder](p: P -> Bool, a: P) {
    is_glb(p, a) implies is_lower_bound(p, a)
}

/// A supremum is an upper bound.
theorem is_lub_is_upper_bound[P: PartialOrder](p: P -> Bool, b: P) {
    is_lub(p, b) implies is_upper_bound(p, b)
}

/// The infimum is above every lower bound.
theorem is_glb_ge_lower_bound[P: PartialOrder](p: P -> Bool, a: P, c: P) {
    is_glb(p, a) and is_lower_bound(p, c) implies c <= a
}

/// The supremum is below every upper bound.
theorem is_lub_le_upper_bound[P: PartialOrder](p: P -> Bool, b: P, c: P) {
    is_lub(p, b) and is_upper_bound(p, c) implies b <= c
}

/// The infimum is below every member of the set.
theorem is_glb_le[P: PartialOrder](p: P -> Bool, a: P, x: P) {
    is_glb(p, a) and p(x) implies a <= x
} by {
    is_lower_bound(p, a)
}

/// The supremum is above every member of the set.
theorem is_lub_ge[P: PartialOrder](p: P -> Bool, b: P, x: P) {
    is_lub(p, b) and p(x) implies x <= b
} by {
    is_upper_bound(p, b)
}

/// An infimum is unique.
theorem is_glb_unique[P: PartialOrder](p: P -> Bool, a: P, a2: P) {
    is_glb(p, a) and is_glb(p, a2) implies a = a2
} by {
    is_lower_bound(p, a)
    is_lower_bound(p, a2)
    a <= a2
    a2 <= a
    lte_antisymm(a, a2)
}

/// A supremum is unique.
theorem is_lub_unique[P: PartialOrder](p: P -> Bool, b: P, b2: P) {
    is_lub(p, b) and is_lub(p, b2) implies b = b2
} by {
    is_upper_bound(p, b)
    is_upper_bound(p, b2)
    b <= b2
    b2 <= b
    lte_antisymm(b, b2)
}

// --- Connections between least/greatest and infimum/supremum ---

/// A least element is the infimum of its set.
theorem is_least_is_glb[P: PartialOrder](p: P -> Bool, a: P) {
    is_least(p, a) implies is_glb(p, a)
} by {
    is_lower_bound(p, a)
    forall(b: P) {
        if is_lower_bound(p, b) {
            p(a)
            b <= a
        }
    }
}

/// A greatest element is the supremum of its set.
theorem is_greatest_is_lub[P: PartialOrder](p: P -> Bool, b: P) {
    is_greatest(p, b) implies is_lub(p, b)
} by {
    is_upper_bound(p, b)
    forall(c: P) {
        if is_upper_bound(p, c) {
            p(b)
            b <= c
        }
    }
}

/// An infimum that belongs to the set is its least element.
theorem is_glb_mem_is_least[P: PartialOrder](p: P -> Bool, a: P) {
    is_glb(p, a) and p(a) implies is_least(p, a)
} by {
    is_lower_bound(p, a)
}

/// A supremum that belongs to the set is its greatest element.
theorem is_lub_mem_is_greatest[P: PartialOrder](p: P -> Bool, b: P) {
    is_lub(p, b) and p(b) implies is_greatest(p, b)
} by {
    is_upper_bound(p, b)
}

// --- Monotonicity of suprema and infima under set inclusion ---

/// Enlarging a set lowers (or fixes) its infimum.
theorem is_glb_mono[P: PartialOrder](p: P -> Bool, q: P -> Bool, a: P, a2: P) {
    is_glb(p, a) and is_glb(q, a2) and forall(x: P) { p(x) implies q(x) }
        implies a2 <= a
} by {
    is_lower_bound(q, a2)
    lower_bound_mono(p, q, a2)
    is_lower_bound(p, a2)
}

/// Enlarging a set raises (or fixes) its supremum.
theorem is_lub_mono[P: PartialOrder](p: P -> Bool, q: P -> Bool, b: P, b2: P) {
    is_lub(p, b) and is_lub(q, b2) and forall(x: P) { p(x) implies q(x) }
        implies b <= b2
} by {
    is_upper_bound(q, b2)
    upper_bound_mono(p, q, b2)
    is_upper_bound(p, b2)
}

// --- The infimum as the greatest lower bound, made explicit ---

/// The infimum is the greatest element of the set of lower bounds.
theorem is_glb_is_greatest_lower_bound[P: PartialOrder](p: P -> Bool, a: P) {
    is_glb(p, a) implies is_greatest(is_lower_bound(p), a)
} by {
    is_lower_bound(p, a)
    let is_in: P -> Bool = is_lower_bound(p)
    is_in(a)
    forall(c: P) {
        if is_in(c) {
            is_lower_bound(p, c)
            c <= a
        }
    }
    is_upper_bound(is_in, a)
}

/// The supremum is the least element of the set of upper bounds.
theorem is_lub_is_least_upper_bound[P: PartialOrder](p: P -> Bool, b: P) {
    is_lub(p, b) implies is_least(is_upper_bound(p), b)
} by {
    is_upper_bound(p, b)
    let is_in: P -> Bool = is_upper_bound(p)
    is_in(b)
    forall(c: P) {
        if is_in(c) {
            is_upper_bound(p, c)
            b <= c
        }
    }
    is_lower_bound(is_in, b)
}

// --- Singletons ---

/// True if `x` equals `a`, standing for the one-element set `{a}`.
define is_singleton[P: PartialOrder](a: P, x: P) -> Bool {
    x = a
}

/// A lower bound of a singleton is below its single element.
theorem singleton_lower_bound_iff[P: PartialOrder](a: P, c: P) {
    is_lower_bound(is_singleton(a), c) = (c <= a)
} by {
    if is_lower_bound(is_singleton(a), c) {
        is_singleton(a, a)
        c <= a
    }
    if c <= a {
        forall(x: P) {
            if is_singleton(a, x) {
                x = a
                c <= x
            }
        }
        is_lower_bound(is_singleton(a), c)
    }
}

/// An upper bound of a singleton is above its single element.
theorem singleton_upper_bound_iff[P: PartialOrder](a: P, c: P) {
    is_upper_bound(is_singleton(a), c) = (a <= c)
} by {
    if is_upper_bound(is_singleton(a), c) {
        is_singleton(a, a)
        a <= c
    }
    if a <= c {
        forall(x: P) {
            if is_singleton(a, x) {
                x = a
                x <= c
            }
        }
        is_upper_bound(is_singleton(a), c)
    }
}

/// The single element of a singleton is its least element.
theorem singleton_is_least[P: PartialOrder](a: P) {
    is_least(is_singleton(a), a)
} by {
    is_singleton(a, a)
    singleton_lower_bound_iff(a, a)
    is_lower_bound(is_singleton(a), a)
}

/// The single element of a singleton is its greatest element.
theorem singleton_is_greatest[P: PartialOrder](a: P) {
    is_greatest(is_singleton(a), a)
} by {
    is_singleton(a, a)
    singleton_upper_bound_iff(a, a)
    is_upper_bound(is_singleton(a), a)
}

/// The single element of a singleton is its infimum.
theorem singleton_is_glb[P: PartialOrder](a: P) {
    is_glb(is_singleton(a), a)
} by {
    singleton_is_least(a)
    is_least_is_glb(is_singleton(a), a)
}

/// The single element of a singleton is its supremum.
theorem singleton_is_lub[P: PartialOrder](a: P) {
    is_lub(is_singleton(a), a)
} by {
    singleton_is_greatest(a)
    is_greatest_is_lub(is_singleton(a), a)
}

// --- The empty set ---

/// The always-false predicate, standing for the empty set.
define is_empty_set[P: PartialOrder](x: P) -> Bool {
    false
}

/// Every element is a lower bound of the empty set.
theorem empty_set_lower_bound[P: PartialOrder](a: P) {
    is_lower_bound(is_empty_set[P], a)
} by {
    is_lower_bound(is_empty_set[P], a) = forall(x: P) {
        is_empty_set(x) implies a <= x
    }
}

/// Every element is an upper bound of the empty set.
theorem empty_set_upper_bound[P: PartialOrder](b: P) {
    is_upper_bound(is_empty_set[P], b)
} by {
    is_upper_bound(is_empty_set[P], b) = forall(x: P) {
        is_empty_set(x) implies x <= b
    }
}

// --- Bounds of the union of two sets ---

/// True if `x` belongs to `p` or to `q`, standing for the union `p ∪ q`.
define pred_union[P: PartialOrder](p: P -> Bool, q: P -> Bool, x: P) -> Bool {
    p(x) or q(x)
}

/// Each part of a union inherits a lower bound of the whole union.
theorem union_lower_bound_parts[P: PartialOrder](p: P -> Bool, q: P -> Bool, a: P) {
    is_lower_bound(pred_union(p, q), a)
        implies is_lower_bound(p, a) and is_lower_bound(q, a)
} by {
    forall(x: P) {
        if p(x) {
            pred_union(p, q, x)
            a <= x
        }
    }
    is_lower_bound(p, a)
    forall(x: P) {
        if q(x) {
            pred_union(p, q, x)
            a <= x
        }
    }
    is_lower_bound(q, a)
}

/// A common lower bound of both parts is a lower bound of their union.
theorem union_lower_bound_of_parts[P: PartialOrder](p: P -> Bool, q: P -> Bool, a: P) {
    is_lower_bound(p, a) and is_lower_bound(q, a)
        implies is_lower_bound(pred_union(p, q), a)
} by {
    forall(x: P) {
        if pred_union(p, q, x) {
            if p(x) {
                a <= x
            } else {
                q(x)
                a <= x
            }
        }
    }
    is_lower_bound(pred_union(p, q), a)
}

/// A lower bound of a union is exactly a common lower bound of both parts.
theorem union_lower_bound_iff_parts[P: PartialOrder](p: P -> Bool, q: P -> Bool, a: P) {
    is_lower_bound(pred_union(p, q), a) iff (is_lower_bound(p, a) and is_lower_bound(q, a))
} by {
    if is_lower_bound(pred_union(p, q), a) {
        union_lower_bound_parts(p, q, a)
    }
    if is_lower_bound(p, a) and is_lower_bound(q, a) {
        union_lower_bound_of_parts(p, q, a)
    }
}

/// Each part of a union inherits an upper bound of the whole union.
theorem union_upper_bound_parts[P: PartialOrder](p: P -> Bool, q: P -> Bool, b: P) {
    is_upper_bound(pred_union(p, q), b)
        implies is_upper_bound(p, b) and is_upper_bound(q, b)
} by {
    forall(x: P) {
        if p(x) {
            pred_union(p, q, x)
            x <= b
        }
    }
    is_upper_bound(p, b)
    forall(x: P) {
        if q(x) {
            pred_union(p, q, x)
            x <= b
        }
    }
    is_upper_bound(q, b)
}

/// A common upper bound of both parts is an upper bound of their union.
theorem union_upper_bound_of_parts[P: PartialOrder](p: P -> Bool, q: P -> Bool, b: P) {
    is_upper_bound(p, b) and is_upper_bound(q, b)
        implies is_upper_bound(pred_union(p, q), b)
} by {
    forall(x: P) {
        if pred_union(p, q, x) {
            if p(x) {
                x <= b
            } else {
                q(x)
                x <= b
            }
        }
    }
    is_upper_bound(pred_union(p, q), b)
}

/// An upper bound of a union is exactly a common upper bound of both parts.
theorem union_upper_bound_iff_parts[P: PartialOrder](p: P -> Bool, q: P -> Bool, b: P) {
    is_upper_bound(pred_union(p, q), b) iff (is_upper_bound(p, b) and is_upper_bound(q, b))
} by {
    if is_upper_bound(pred_union(p, q), b) {
        union_upper_bound_parts(p, q, b)
    }
    if is_upper_bound(p, b) and is_upper_bound(q, b) {
        union_upper_bound_of_parts(p, q, b)
    }
}

/// The infimum of a union of two sets is the meet of their infima.
theorem union_is_glb[S: MeetSemilattice](p: S -> Bool, q: S -> Bool, a: S, b: S) {
    is_glb(p, a) and is_glb(q, b) implies is_glb(pred_union(p, q), a.meet(b))
} by {
    is_lower_bound(p, a)
    is_lower_bound(q, b)
    meet_lte_left(a, b)
    meet_lte_right(a, b)
    lower_bound_of_le(p, a, a.meet(b))
    lower_bound_of_le(q, b, a.meet(b))
    is_lower_bound(p, a.meet(b))
    is_lower_bound(q, a.meet(b))
    union_lower_bound_of_parts(p, q, a.meet(b))
    is_lower_bound(pred_union(p, q), a.meet(b))
    forall(c: S) {
        if is_lower_bound(pred_union(p, q), c) {
            union_lower_bound_parts(p, q, c)
            is_lower_bound(p, c)
            is_lower_bound(q, c)
            c <= a
            c <= b
            lte_meet_of_bounds(c, a, b)
            c <= a.meet(b)
        }
    }
}

/// The supremum of a union of two sets is the join of their suprema.
theorem union_is_lub[S: JoinSemilattice](p: S -> Bool, q: S -> Bool, a: S, b: S) {
    is_lub(p, a) and is_lub(q, b) implies is_lub(pred_union(p, q), a.join(b))
} by {
    is_upper_bound(p, a)
    is_upper_bound(q, b)
    lte_join_left(a, b)
    lte_join_right(a, b)
    upper_bound_of_ge(p, a, a.join(b))
    upper_bound_of_ge(q, b, a.join(b))
    is_upper_bound(p, a.join(b))
    is_upper_bound(q, a.join(b))
    union_upper_bound_of_parts(p, q, a.join(b))
    is_upper_bound(pred_union(p, q), a.join(b))
    forall(c: S) {
        if is_upper_bound(pred_union(p, q), c) {
            union_upper_bound_parts(p, q, c)
            is_upper_bound(p, c)
            is_upper_bound(q, c)
            a <= c
            b <= c
            join_lte_of_bounds(a, b, c)
            a.join(b) <= c
        }
    }
}

// --- Bounds of the intersection of two sets ---

/// True if `x` belongs to both `p` and `q`, standing for the intersection `p ∩ q`.
define pred_inter[P: PartialOrder](p: P -> Bool, q: P -> Bool, x: P) -> Bool {
    p(x) and q(x)
}

/// A lower bound of `p` is a lower bound of any intersection with `p`.
theorem inter_lower_bound_left[P: PartialOrder](p: P -> Bool, q: P -> Bool, a: P) {
    is_lower_bound(p, a) implies is_lower_bound(pred_inter(p, q), a)
} by {
    forall(x: P) {
        if pred_inter(p, q, x) {
            p(x)
            a <= x
        }
    }
}

/// A lower bound of `q` is a lower bound of any intersection with `q`.
theorem inter_lower_bound_right[P: PartialOrder](p: P -> Bool, q: P -> Bool, a: P) {
    is_lower_bound(q, a) implies is_lower_bound(pred_inter(p, q), a)
} by {
    forall(x: P) {
        if pred_inter(p, q, x) {
            q(x)
            a <= x
        }
    }
}

/// An upper bound of `p` is an upper bound of any intersection with `p`.
theorem inter_upper_bound_left[P: PartialOrder](p: P -> Bool, q: P -> Bool, b: P) {
    is_upper_bound(p, b) implies is_upper_bound(pred_inter(p, q), b)
} by {
    forall(x: P) {
        if pred_inter(p, q, x) {
            p(x)
            x <= b
        }
    }
}

/// An upper bound of `q` is an upper bound of any intersection with `q`.
theorem inter_upper_bound_right[P: PartialOrder](p: P -> Bool, q: P -> Bool, b: P) {
    is_upper_bound(q, b) implies is_upper_bound(pred_inter(p, q), b)
} by {
    forall(x: P) {
        if pred_inter(p, q, x) {
            q(x)
            x <= b
        }
    }
}

// --- Insertion of a single element ---

/// True if `x` equals `a` or already lies in `p`, standing for `{a} ∪ p`.
define pred_insert[P: PartialOrder](a: P, p: P -> Bool, x: P) -> Bool {
    x = a or p(x)
}

/// A lower bound of `pred_insert(a, p)` is below `a`.
theorem insert_lower_bound_le[P: PartialOrder](a: P, p: P -> Bool, c: P) {
    is_lower_bound(pred_insert(a, p), c) implies c <= a
} by {
    pred_insert(a, p, a)
    lower_bound_le(pred_insert(a, p), c, a)
}

/// A lower bound of `pred_insert(a, p)` is a lower bound of `p`.
theorem insert_lower_bound_rest[P: PartialOrder](a: P, p: P -> Bool, c: P) {
    is_lower_bound(pred_insert(a, p), c) implies is_lower_bound(p, c)
} by {
    forall(x: P) {
        if p(x) {
            pred_insert(a, p, x)
            c <= x
        }
    }
}

/// Combining a lower bound of `a` with a lower bound of `p` gives a lower bound of `pred_insert(a, p)`.
theorem insert_lower_bound_of[P: PartialOrder](a: P, p: P -> Bool, c: P) {
    c <= a and is_lower_bound(p, c) implies is_lower_bound(pred_insert(a, p), c)
} by {
    forall(x: P) {
        if pred_insert(a, p, x) {
            if x = a {
                c <= x
            } else {
                p(x)
                c <= x
            }
        }
    }
}

/// A lower bound of an insertion is exactly a bound below the inserted point and the rest.
theorem insert_lower_bound_iff[P: PartialOrder](a: P, p: P -> Bool, c: P) {
    is_lower_bound(pred_insert(a, p), c) iff (c <= a and is_lower_bound(p, c))
} by {
    if is_lower_bound(pred_insert(a, p), c) {
        insert_lower_bound_le(a, p, c)
        c <= a
        insert_lower_bound_rest(a, p, c)
        is_lower_bound(p, c)
        c <= a and is_lower_bound(p, c)
    }
    if c <= a and is_lower_bound(p, c) {
        insert_lower_bound_of(a, p, c)
        is_lower_bound(pred_insert(a, p), c)
    }
}

/// An upper bound of `pred_insert(a, p)` is above `a`.
theorem insert_upper_bound_ge[P: PartialOrder](a: P, p: P -> Bool, c: P) {
    is_upper_bound(pred_insert(a, p), c) implies a <= c
} by {
    pred_insert(a, p, a)
    upper_bound_ge(pred_insert(a, p), c, a)
}

/// An upper bound of `pred_insert(a, p)` is an upper bound of `p`.
theorem insert_upper_bound_rest[P: PartialOrder](a: P, p: P -> Bool, c: P) {
    is_upper_bound(pred_insert(a, p), c) implies is_upper_bound(p, c)
} by {
    forall(x: P) {
        if p(x) {
            pred_insert(a, p, x)
            x <= c
        }
    }
}

/// Combining an upper bound of `a` with an upper bound of `p` gives an upper bound of `pred_insert(a, p)`.
theorem insert_upper_bound_of[P: PartialOrder](a: P, p: P -> Bool, c: P) {
    a <= c and is_upper_bound(p, c) implies is_upper_bound(pred_insert(a, p), c)
} by {
    forall(x: P) {
        if pred_insert(a, p, x) {
            if x = a {
                x <= c
            } else {
                p(x)
                x <= c
            }
        }
    }
}

/// An upper bound of an insertion is exactly a bound above the inserted point and the rest.
theorem insert_upper_bound_iff[P: PartialOrder](a: P, p: P -> Bool, c: P) {
    is_upper_bound(pred_insert(a, p), c) iff (a <= c and is_upper_bound(p, c))
} by {
    if is_upper_bound(pred_insert(a, p), c) {
        insert_upper_bound_ge(a, p, c)
        a <= c
        insert_upper_bound_rest(a, p, c)
        is_upper_bound(p, c)
        a <= c and is_upper_bound(p, c)
    }
    if a <= c and is_upper_bound(p, c) {
        insert_upper_bound_of(a, p, c)
        is_upper_bound(pred_insert(a, p), c)
    }
}

// --- Meet and join as binary infima and suprema ---

/// True if `x` is one of the two elements `a` or `b`.
define is_in_pair[P: PartialOrder](a: P, b: P, x: P) -> Bool {
    x = a or x = b
}

/// The meet is a lower bound of the two-element set.
theorem meet_is_lower_bound[S: MeetSemilattice](a: S, b: S) {
    is_lower_bound(is_in_pair(a, b), a.meet(b))
} by {
    forall(x: S) {
        if is_in_pair(a, b, x) {
            if x = a {
                meet_lte_left(a, b)
                a.meet(b) <= x
            } else {
                x = b
                meet_lte_right(a, b)
                a.meet(b) <= x
            }
        }
    }
}

/// The join is an upper bound of the two-element set.
theorem join_is_upper_bound[S: JoinSemilattice](a: S, b: S) {
    is_upper_bound(is_in_pair(a, b), a.join(b))
} by {
    forall(x: S) {
        if is_in_pair(a, b, x) {
            if x = a {
                lte_join_left(a, b)
                x <= a.join(b)
            } else {
                x = b
                lte_join_right(a, b)
                x <= a.join(b)
            }
        }
    }
}

/// The meet of two elements is the infimum of the two-element set.
theorem meet_is_glb[S: MeetSemilattice](a: S, b: S) {
    is_glb(is_in_pair(a, b), a.meet(b))
} by {
    meet_is_lower_bound(a, b)
    forall(c: S) {
        if is_lower_bound(is_in_pair(a, b), c) {
            is_in_pair(a, b, a)
            is_in_pair(a, b, b)
            c <= a
            c <= b
            lte_meet_of_bounds(c, a, b)
            c <= a.meet(b)
        }
    }
}

/// The join of two elements is the supremum of the two-element set.
theorem join_is_lub[S: JoinSemilattice](a: S, b: S) {
    is_lub(is_in_pair(a, b), a.join(b))
} by {
    join_is_upper_bound(a, b)
    forall(c: S) {
        if is_upper_bound(is_in_pair(a, b), c) {
            is_in_pair(a, b, a)
            is_in_pair(a, b, b)
            a <= c
            b <= c
            join_lte_of_bounds(a, b, c)
            a.join(b) <= c
        }
    }
}

/// The infimum of `pred_insert(a, p)` is the meet of `a` with the infimum of `p`.
theorem insert_is_glb[S: MeetSemilattice](a: S, p: S -> Bool, g: S) {
    is_glb(p, g) implies is_glb(pred_insert(a, p), a.meet(g))
} by {
    is_lower_bound(p, g)
    meet_lte_left(a, g)
    meet_lte_right(a, g)
    lower_bound_of_le(p, g, a.meet(g))
    is_lower_bound(p, a.meet(g))
    insert_lower_bound_of(a, p, a.meet(g))
    is_lower_bound(pred_insert(a, p), a.meet(g))
    forall(c: S) {
        if is_lower_bound(pred_insert(a, p), c) {
            insert_lower_bound_le(a, p, c)
            insert_lower_bound_rest(a, p, c)
            c <= a
            is_lower_bound(p, c)
            c <= g
            lte_meet_of_bounds(c, a, g)
            c <= a.meet(g)
        }
    }
}

/// The supremum of `pred_insert(a, p)` is the join of `a` with the supremum of `p`.
theorem insert_is_lub[S: JoinSemilattice](a: S, p: S -> Bool, g: S) {
    is_lub(p, g) implies is_lub(pred_insert(a, p), a.join(g))
} by {
    is_upper_bound(p, g)
    lte_join_left(a, g)
    lte_join_right(a, g)
    upper_bound_of_ge(p, g, a.join(g))
    is_upper_bound(p, a.join(g))
    insert_upper_bound_of(a, p, a.join(g))
    is_upper_bound(pred_insert(a, p), a.join(g))
    forall(c: S) {
        if is_upper_bound(pred_insert(a, p), c) {
            insert_upper_bound_ge(a, p, c)
            insert_upper_bound_rest(a, p, c)
            a <= c
            is_upper_bound(p, c)
            g <= c
            join_lte_of_bounds(a, g, c)
            a.join(g) <= c
        }
    }
}

// --- Boundedness of a set ---

/// True if some element is an upper bound of `p`.
define is_bounded_above[P: PartialOrder](p: P -> Bool) -> Bool {
    exists(b: P) { is_upper_bound(p, b) }
}

/// True if some element is a lower bound of `p`.
define is_bounded_below[P: PartialOrder](p: P -> Bool) -> Bool {
    exists(a: P) { is_lower_bound(p, a) }
}

/// An explicit upper bound witnesses bounded above.
theorem is_bounded_above_of[P: PartialOrder](p: P -> Bool, b: P) {
    is_upper_bound(p, b) implies is_bounded_above(p)
}

/// An explicit lower bound witnesses bounded below.
theorem is_bounded_below_of[P: PartialOrder](p: P -> Bool, a: P) {
    is_lower_bound(p, a) implies is_bounded_below(p)
}

/// Unfold lemma for `is_bounded_above`.
theorem is_bounded_above_iff[P: PartialOrder](p: P -> Bool) {
    is_bounded_above(p) iff exists(b: P) { is_upper_bound(p, b) }
}

/// Unfold lemma for `is_bounded_below`.
theorem is_bounded_below_iff[P: PartialOrder](p: P -> Bool) {
    is_bounded_below(p) iff exists(a: P) { is_lower_bound(p, a) }
}

/// A supremum witnesses bounded above.
theorem is_lub_imp_bounded_above[P: PartialOrder](p: P -> Bool, b: P) {
    is_lub(p, b) implies is_bounded_above(p)
} by {
    is_upper_bound(p, b)
}

/// An infimum witnesses bounded below.
theorem is_glb_imp_bounded_below[P: PartialOrder](p: P -> Bool, a: P) {
    is_glb(p, a) implies is_bounded_below(p)
} by {
    is_lower_bound(p, a)
}

/// A greatest element witnesses bounded above.
theorem is_greatest_imp_bounded_above[P: PartialOrder](p: P -> Bool, b: P) {
    is_greatest(p, b) implies is_bounded_above(p)
} by {
    is_upper_bound(p, b)
}

/// A least element witnesses bounded below.
theorem is_least_imp_bounded_below[P: PartialOrder](p: P -> Bool, a: P) {
    is_least(p, a) implies is_bounded_below(p)
} by {
    is_lower_bound(p, a)
}

/// A subset of a bounded-above set is bounded above.
theorem bounded_above_mono[P: PartialOrder](p: P -> Bool, q: P -> Bool) {
    is_bounded_above(q) and forall(x: P) { p(x) implies q(x) } implies is_bounded_above(p)
} by {
    let b: P satisfy { is_upper_bound(q, b) }
    upper_bound_mono(p, q, b)
    is_upper_bound(p, b)
}

/// A subset of a bounded-below set is bounded below.
theorem bounded_below_mono[P: PartialOrder](p: P -> Bool, q: P -> Bool) {
    is_bounded_below(q) and forall(x: P) { p(x) implies q(x) } implies is_bounded_below(p)
} by {
    let a: P satisfy { is_lower_bound(q, a) }
    lower_bound_mono(p, q, a)
    is_lower_bound(p, a)
}

/// The empty set is bounded above.
theorem empty_set_bounded_above[P: PartialOrder](b: P) {
    is_bounded_above(is_empty_set[P])
} by {
    empty_set_upper_bound(b)
    is_upper_bound(is_empty_set[P], b)
}

/// The empty set is bounded below.
theorem empty_set_bounded_below[P: PartialOrder](a: P) {
    is_bounded_below(is_empty_set[P])
} by {
    empty_set_lower_bound(a)
    is_lower_bound(is_empty_set[P], a)
}

/// A singleton is bounded above.
theorem singleton_bounded_above[P: PartialOrder](a: P) {
    is_bounded_above(is_singleton(a))
} by {
    singleton_is_greatest(a)
    is_upper_bound(is_singleton(a), a)
}

/// A singleton is bounded below.
theorem singleton_bounded_below[P: PartialOrder](a: P) {
    is_bounded_below(is_singleton(a))
} by {
    singleton_is_least(a)
    is_lower_bound(is_singleton(a), a)
}

// --- Boundedness of unions, intersections, and insertions ---

/// The union of two sets bounded above by a common bound is bounded above.
theorem union_bounded_above_of_common[P: PartialOrder](p: P -> Bool, q: P -> Bool, b: P) {
    is_upper_bound(p, b) and is_upper_bound(q, b) implies is_bounded_above(pred_union(p, q))
} by {
    union_upper_bound_of_parts(p, q, b)
    is_upper_bound(pred_union(p, q), b)
}

/// The union of two sets bounded below by a common bound is bounded below.
theorem union_bounded_below_of_common[P: PartialOrder](p: P -> Bool, q: P -> Bool, a: P) {
    is_lower_bound(p, a) and is_lower_bound(q, a) implies is_bounded_below(pred_union(p, q))
} by {
    union_lower_bound_of_parts(p, q, a)
    is_lower_bound(pred_union(p, q), a)
}

/// A subset of `p` (`pred_inter` with anything) inherits boundedness above from `p`.
theorem inter_bounded_above_left[P: PartialOrder](p: P -> Bool, q: P -> Bool) {
    is_bounded_above(p) implies is_bounded_above(pred_inter(p, q))
} by {
    let b: P satisfy { is_upper_bound(p, b) }
    inter_upper_bound_left(p, q, b)
    is_upper_bound(pred_inter(p, q), b)
}

/// A subset of `q` (`pred_inter` with anything) inherits boundedness above from `q`.
theorem inter_bounded_above_right[P: PartialOrder](p: P -> Bool, q: P -> Bool) {
    is_bounded_above(q) implies is_bounded_above(pred_inter(p, q))
} by {
    let b: P satisfy { is_upper_bound(q, b) }
    inter_upper_bound_right(p, q, b)
    is_upper_bound(pred_inter(p, q), b)
}

/// A subset of `p` (`pred_inter` with anything) inherits boundedness below from `p`.
theorem inter_bounded_below_left[P: PartialOrder](p: P -> Bool, q: P -> Bool) {
    is_bounded_below(p) implies is_bounded_below(pred_inter(p, q))
} by {
    let a: P satisfy { is_lower_bound(p, a) }
    inter_lower_bound_left(p, q, a)
    is_lower_bound(pred_inter(p, q), a)
}

/// A subset of `q` (`pred_inter` with anything) inherits boundedness below from `q`.
theorem inter_bounded_below_right[P: PartialOrder](p: P -> Bool, q: P -> Bool) {
    is_bounded_below(q) implies is_bounded_below(pred_inter(p, q))
} by {
    let a: P satisfy { is_lower_bound(q, a) }
    inter_lower_bound_right(p, q, a)
    is_lower_bound(pred_inter(p, q), a)
}

/// An explicit upper bound for `a` and for `p` witnesses that `pred_insert(a, p)` is bounded above.
theorem insert_bounded_above_of[P: PartialOrder](a: P, p: P -> Bool, c: P) {
    a <= c and is_upper_bound(p, c) implies is_bounded_above(pred_insert(a, p))
} by {
    insert_upper_bound_of(a, p, c)
    is_upper_bound(pred_insert(a, p), c)
}

/// An explicit lower bound for `a` and for `p` witnesses that `pred_insert(a, p)` is bounded below.
theorem insert_bounded_below_of[P: PartialOrder](a: P, p: P -> Bool, c: P) {
    c <= a and is_lower_bound(p, c) implies is_bounded_below(pred_insert(a, p))
} by {
    insert_lower_bound_of(a, p, c)
    is_lower_bound(pred_insert(a, p), c)
}

/// In a join semilattice, inserting an element preserves boundedness above.
theorem insert_bounded_above[S: JoinSemilattice](a: S, p: S -> Bool) {
    is_bounded_above(p) implies is_bounded_above(pred_insert(a, p))
} by {
    let c: S satisfy { is_upper_bound(p, c) }
    lte_join_left(a, c)
    lte_join_right(a, c)
    upper_bound_of_ge(p, c, a.join(c))
    is_upper_bound(p, a.join(c))
    insert_bounded_above_of(a, p, a.join(c))
}

/// In a meet semilattice, inserting an element preserves boundedness below.
theorem insert_bounded_below[S: MeetSemilattice](a: S, p: S -> Bool) {
    is_bounded_below(p) implies is_bounded_below(pred_insert(a, p))
} by {
    let c: S satisfy { is_lower_bound(p, c) }
    meet_lte_left(a, c)
    meet_lte_right(a, c)
    lower_bound_of_le(p, c, a.meet(c))
    is_lower_bound(p, a.meet(c))
    insert_bounded_below_of(a, p, a.meet(c))
}

/// In a join semilattice, the union of two bounded-above sets is bounded above.
theorem union_bounded_above[S: JoinSemilattice](p: S -> Bool, q: S -> Bool) {
    is_bounded_above(p) and is_bounded_above(q) implies is_bounded_above(pred_union(p, q))
} by {
    let b1: S satisfy { is_upper_bound(p, b1) }
    let b2: S satisfy { is_upper_bound(q, b2) }
    lte_join_left(b1, b2)
    lte_join_right(b1, b2)
    upper_bound_of_ge(p, b1, b1.join(b2))
    upper_bound_of_ge(q, b2, b1.join(b2))
    union_bounded_above_of_common(p, q, b1.join(b2))
}

/// In a meet semilattice, the union of two bounded-below sets is bounded below.
theorem union_bounded_below[S: MeetSemilattice](p: S -> Bool, q: S -> Bool) {
    is_bounded_below(p) and is_bounded_below(q) implies is_bounded_below(pred_union(p, q))
} by {
    let a1: S satisfy { is_lower_bound(p, a1) }
    let a2: S satisfy { is_lower_bound(q, a2) }
    meet_lte_left(a1, a2)
    meet_lte_right(a1, a2)
    lower_bound_of_le(p, a1, a1.meet(a2))
    lower_bound_of_le(q, a2, a1.meet(a2))
    union_bounded_below_of_common(p, q, a1.meet(a2))
}

// --- Transport of bounds along predicate equality ---

/// A lower bound transports along pointwise predicate equality.
theorem is_lower_bound_of_pred_eq[P: PartialOrder](p: P -> Bool, q: P -> Bool, a: P) {
    is_lower_bound(p, a) and forall(x: P) { p(x) iff q(x) } implies is_lower_bound(q, a)
} by {
    forall(x: P) {
        if q(x) {
            p(x)
            a <= x
        }
    }
}

/// An upper bound transports along pointwise predicate equality.
theorem is_upper_bound_of_pred_eq[P: PartialOrder](p: P -> Bool, q: P -> Bool, b: P) {
    is_upper_bound(p, b) and forall(x: P) { p(x) iff q(x) } implies is_upper_bound(q, b)
} by {
    forall(x: P) {
        if q(x) {
            p(x)
            x <= b
        }
    }
}

/// The infimum transports along pointwise predicate equality.
theorem is_glb_of_pred_eq[P: PartialOrder](p: P -> Bool, q: P -> Bool, a: P) {
    is_glb(p, a) and forall(x: P) { p(x) iff q(x) } implies is_glb(q, a)
} by {
    is_lower_bound(p, a)
    is_lower_bound_of_pred_eq(p, q, a)
    is_lower_bound(q, a)
    forall(c: P) {
        if is_lower_bound(q, c) {
            is_lower_bound_of_pred_eq(q, p, c)
            is_lower_bound(p, c)
            c <= a
        }
    }
}

/// The supremum transports along pointwise predicate equality.
theorem is_lub_of_pred_eq[P: PartialOrder](p: P -> Bool, q: P -> Bool, b: P) {
    is_lub(p, b) and forall(x: P) { p(x) iff q(x) } implies is_lub(q, b)
} by {
    is_upper_bound(p, b)
    is_upper_bound_of_pred_eq(p, q, b)
    is_upper_bound(q, b)
    forall(c: P) {
        if is_upper_bound(q, c) {
            is_upper_bound_of_pred_eq(q, p, c)
            is_upper_bound(p, c)
            b <= c
        }
    }
}

/// The least element transports along pointwise predicate equality.
theorem is_least_of_pred_eq[P: PartialOrder](p: P -> Bool, q: P -> Bool, a: P) {
    is_least(p, a) and forall(x: P) { p(x) iff q(x) } implies is_least(q, a)
} by {
    p(a)
    q(a)
    is_lower_bound(p, a)
    is_lower_bound_of_pred_eq(p, q, a)
    is_lower_bound(q, a)
}

/// The greatest element transports along pointwise predicate equality.
theorem is_greatest_of_pred_eq[P: PartialOrder](p: P -> Bool, q: P -> Bool, b: P) {
    is_greatest(p, b) and forall(x: P) { p(x) iff q(x) } implies is_greatest(q, b)
} by {
    p(b)
    q(b)
    is_upper_bound(p, b)
    is_upper_bound_of_pred_eq(p, q, b)
    is_upper_bound(q, b)
}

/// Lower-bound predicate-equality transport in iff form.
theorem is_lower_bound_pred_eq_iff[P: PartialOrder](p: P -> Bool, q: P -> Bool, a: P) {
    forall(x: P) { p(x) iff q(x) } implies (is_lower_bound(p, a) iff is_lower_bound(q, a))
} by {
    if forall(x: P) { p(x) iff q(x) } {
        if is_lower_bound(p, a) {
            is_lower_bound_of_pred_eq(p, q, a)
        }
        if is_lower_bound(q, a) {
            is_lower_bound_of_pred_eq(q, p, a)
        }
    }
}

/// Upper-bound predicate-equality transport in iff form.
theorem is_upper_bound_pred_eq_iff[P: PartialOrder](p: P -> Bool, q: P -> Bool, b: P) {
    forall(x: P) { p(x) iff q(x) } implies (is_upper_bound(p, b) iff is_upper_bound(q, b))
} by {
    if forall(x: P) { p(x) iff q(x) } {
        if is_upper_bound(p, b) {
            is_upper_bound_of_pred_eq(p, q, b)
        }
        if is_upper_bound(q, b) {
            is_upper_bound_of_pred_eq(q, p, b)
        }
    }
}

/// Infimum predicate-equality transport in iff form.
theorem is_glb_pred_eq_iff[P: PartialOrder](p: P -> Bool, q: P -> Bool, a: P) {
    forall(x: P) { p(x) iff q(x) } implies (is_glb(p, a) iff is_glb(q, a))
} by {
    if forall(x: P) { p(x) iff q(x) } {
        if is_glb(p, a) {
            is_glb_of_pred_eq(p, q, a)
        }
        if is_glb(q, a) {
            is_glb_of_pred_eq(q, p, a)
        }
    }
}

/// Supremum predicate-equality transport in iff form.
theorem is_lub_pred_eq_iff[P: PartialOrder](p: P -> Bool, q: P -> Bool, b: P) {
    forall(x: P) { p(x) iff q(x) } implies (is_lub(p, b) iff is_lub(q, b))
} by {
    if forall(x: P) { p(x) iff q(x) } {
        if is_lub(p, b) {
            is_lub_of_pred_eq(p, q, b)
        }
        if is_lub(q, b) {
            is_lub_of_pred_eq(q, p, b)
        }
    }
}

/// Least-element predicate-equality transport in iff form.
theorem is_least_pred_eq_iff[P: PartialOrder](p: P -> Bool, q: P -> Bool, a: P) {
    forall(x: P) { p(x) iff q(x) } implies (is_least(p, a) iff is_least(q, a))
} by {
    if forall(x: P) { p(x) iff q(x) } {
        if is_least(p, a) {
            is_least_of_pred_eq(p, q, a)
        }
        if is_least(q, a) {
            is_least_of_pred_eq(q, p, a)
        }
    }
}

/// Greatest-element predicate-equality transport in iff form.
theorem is_greatest_pred_eq_iff[P: PartialOrder](p: P -> Bool, q: P -> Bool, b: P) {
    forall(x: P) { p(x) iff q(x) } implies (is_greatest(p, b) iff is_greatest(q, b))
} by {
    if forall(x: P) { p(x) iff q(x) } {
        if is_greatest(p, b) {
            is_greatest_of_pred_eq(p, q, b)
        }
        if is_greatest(q, b) {
            is_greatest_of_pred_eq(q, p, b)
        }
    }
}

/// Boundedness above transports along pointwise predicate equality.
theorem is_bounded_above_of_pred_eq[P: PartialOrder](p: P -> Bool, q: P -> Bool) {
    is_bounded_above(p) and forall(x: P) { p(x) iff q(x) } implies is_bounded_above(q)
} by {
    let b: P satisfy { is_upper_bound(p, b) }
    is_upper_bound_of_pred_eq(p, q, b)
    is_upper_bound(q, b)
}

/// Boundedness below transports along pointwise predicate equality.
theorem is_bounded_below_of_pred_eq[P: PartialOrder](p: P -> Bool, q: P -> Bool) {
    is_bounded_below(p) and forall(x: P) { p(x) iff q(x) } implies is_bounded_below(q)
} by {
    let a: P satisfy { is_lower_bound(p, a) }
    is_lower_bound_of_pred_eq(p, q, a)
    is_lower_bound(q, a)
}

/// Boundedness above predicate-equality transport in iff form.
theorem is_bounded_above_pred_eq_iff[P: PartialOrder](p: P -> Bool, q: P -> Bool) {
    forall(x: P) { p(x) iff q(x) } implies (is_bounded_above(p) iff is_bounded_above(q))
} by {
    if forall(x: P) { p(x) iff q(x) } {
        if is_bounded_above(p) {
            is_bounded_above_of_pred_eq(p, q)
        }
        if is_bounded_above(q) {
            is_bounded_above_of_pred_eq(q, p)
        }
    }
}

/// Boundedness below predicate-equality transport in iff form.
theorem is_bounded_below_pred_eq_iff[P: PartialOrder](p: P -> Bool, q: P -> Bool) {
    forall(x: P) { p(x) iff q(x) } implies (is_bounded_below(p) iff is_bounded_below(q))
} by {
    if forall(x: P) { p(x) iff q(x) } {
        if is_bounded_below(p) {
            is_bounded_below_of_pred_eq(p, q)
        }
        if is_bounded_below(q) {
            is_bounded_below_of_pred_eq(q, p)
        }
    }
}

// --- Two-sided boundedness ---

/// True if `p` is bounded both above and below.
define is_bounded[P: PartialOrder](p: P -> Bool) -> Bool {
    is_bounded_above(p) and is_bounded_below(p)
}

/// Unfold lemma for `is_bounded`.
theorem is_bounded_iff[P: PartialOrder](p: P -> Bool) {
    is_bounded(p) iff (is_bounded_above(p) and is_bounded_below(p))
}

/// Assemble `is_bounded` from its two halves.
theorem is_bounded_intro[P: PartialOrder](p: P -> Bool) {
    is_bounded_above(p) and is_bounded_below(p) implies is_bounded(p)
}

/// A two-sided bounded set is bounded above.
theorem is_bounded_imp_bounded_above[P: PartialOrder](p: P -> Bool) {
    is_bounded(p) implies is_bounded_above(p)
} by {
    is_bounded_iff(p)
}

/// A two-sided bounded set is bounded below.
theorem is_bounded_imp_bounded_below[P: PartialOrder](p: P -> Bool) {
    is_bounded(p) implies is_bounded_below(p)
} by {
    is_bounded_iff(p)
}

/// Two-sided boundedness from explicit lower and upper bounds.
theorem is_bounded_of[P: PartialOrder](p: P -> Bool, a: P, b: P) {
    is_lower_bound(p, a) and is_upper_bound(p, b) implies is_bounded(p)
} by {
    is_bounded_below_of(p, a)
    is_bounded_above_of(p, b)
    is_bounded_intro(p)
}

/// The empty set is two-sided bounded.
theorem empty_set_bounded[P: PartialOrder](a: P) {
    is_bounded(is_empty_set[P])
} by {
    empty_set_bounded_above(a)
    empty_set_bounded_below(a)
    is_bounded_intro(is_empty_set[P])
}

/// A singleton is two-sided bounded.
theorem singleton_bounded[P: PartialOrder](a: P) {
    is_bounded(is_singleton(a))
} by {
    singleton_bounded_above(a)
    singleton_bounded_below(a)
    is_bounded_intro(is_singleton(a))
}

/// Two-sided boundedness transports along pointwise predicate equality.
theorem is_bounded_of_pred_eq[P: PartialOrder](p: P -> Bool, q: P -> Bool) {
    (forall(x: P) { p(x) iff q(x) }) and is_bounded(p) implies is_bounded(q)
} by {
    if (forall(x: P) { p(x) iff q(x) }) and is_bounded(p) {
        is_bounded_iff(p)
        is_bounded_above_of_pred_eq(p, q)
        is_bounded_below_of_pred_eq(p, q)
        is_bounded_intro(q)
    }
}

/// Two-sided boundedness predicate-equality transport in iff form.
theorem is_bounded_pred_eq_iff[P: PartialOrder](p: P -> Bool, q: P -> Bool) {
    (forall(x: P) { p(x) iff q(x) }) implies (is_bounded(p) iff is_bounded(q))
} by {
    if forall(x: P) { p(x) iff q(x) } {
        is_bounded_above_pred_eq_iff(p, q)
        is_bounded_below_pred_eq_iff(p, q)
        is_bounded_iff(p)
        is_bounded_iff(q)
    }
}

/// In a lattice, a two-element predicate `is_in_pair(a, b)` is two-sided bounded.
theorem pair_bounded[L: Lattice](a: L, b: L) {
    is_bounded(is_in_pair(a, b))
} by {
    meet_is_lower_bound(a, b)
    join_is_upper_bound(a, b)
    is_bounded_below_of(is_in_pair(a, b), a.meet(b))
    is_bounded_above_of(is_in_pair(a, b), a.join(b))
    is_bounded_intro(is_in_pair(a, b))
}

/// In a join semilattice, a two-element predicate `is_in_pair(a, b)` is bounded above.
theorem pair_bounded_above[S: JoinSemilattice](a: S, b: S) {
    is_bounded_above(is_in_pair(a, b))
} by {
    join_is_upper_bound(a, b)
    is_bounded_above_of(is_in_pair(a, b), a.join(b))
}

/// In a meet semilattice, a two-element predicate `is_in_pair(a, b)` is bounded below.
theorem pair_bounded_below[S: MeetSemilattice](a: S, b: S) {
    is_bounded_below(is_in_pair(a, b))
} by {
    meet_is_lower_bound(a, b)
    is_bounded_below_of(is_in_pair(a, b), a.meet(b))
}

/// A lower bound of `is_in_pair(a, b)` is a common lower bound of `a` and `b`.
theorem is_lower_bound_is_in_pair_imp[P: PartialOrder](a: P, b: P, c: P) {
    is_lower_bound(is_in_pair(a, b), c) implies (c <= a and c <= b)
} by {
    if is_lower_bound(is_in_pair(a, b), c) {
        is_in_pair(a, b, a)
        is_in_pair(a, b, b)
    }
}

/// A common lower bound of `a` and `b` is a lower bound of `is_in_pair(a, b)`.
theorem is_lower_bound_is_in_pair_of[P: PartialOrder](a: P, b: P, c: P) {
    c <= a and c <= b implies is_lower_bound(is_in_pair(a, b), c)
} by {
    if c <= a and c <= b {
        forall(x: P) {
            if is_in_pair(a, b, x) {
                if x = a {
                    c <= x
                } else {
                    x = b
                    c <= x
                }
            }
        }
    }
}

/// An upper bound of `is_in_pair(a, b)` is a common upper bound of `a` and `b`.
theorem is_upper_bound_is_in_pair_imp[P: PartialOrder](a: P, b: P, c: P) {
    is_upper_bound(is_in_pair(a, b), c) implies (a <= c and b <= c)
} by {
    if is_upper_bound(is_in_pair(a, b), c) {
        is_in_pair(a, b, a)
        is_in_pair(a, b, b)
    }
}

/// A common upper bound of `a` and `b` is an upper bound of `is_in_pair(a, b)`.
theorem is_upper_bound_is_in_pair_of[P: PartialOrder](a: P, b: P, c: P) {
    a <= c and b <= c implies is_upper_bound(is_in_pair(a, b), c)
} by {
    if a <= c and b <= c {
        forall(x: P) {
            if is_in_pair(a, b, x) {
                if x = a {
                    x <= c
                } else {
                    x = b
                    x <= c
                }
            }
        }
    }
}

/// `is_in_pair` is symmetric in its arguments.
theorem is_in_pair_comm[P: PartialOrder](a: P, b: P, x: P) {
    is_in_pair(a, b, x) iff is_in_pair(b, a, x)
} by {
    if is_in_pair(a, b, x) {
        x = a or x = b
        is_in_pair(b, a, x)
    }
    if is_in_pair(b, a, x) {
        x = b or x = a
        is_in_pair(a, b, x)
    }
}

/// `is_in_pair(a, a, x)` holds exactly when `x = a`.
theorem is_in_pair_self_iff[P: PartialOrder](a: P, x: P) {
    is_in_pair(a, a, x) iff x = a
}

/// Each component is in the pair.
theorem is_in_pair_left[P: PartialOrder](a: P, b: P) {
    is_in_pair(a, b, a)
}

/// Each component is in the pair.
theorem is_in_pair_right[P: PartialOrder](a: P, b: P) {
    is_in_pair(a, b, b)
}

/// `is_in_pair` unfolds to a disjunction of equalities.
theorem is_in_pair_iff[P: PartialOrder](a: P, b: P, x: P) {
    is_in_pair(a, b, x) iff (x = a or x = b)
}

/// `x = a` implies `is_in_pair(a, b, x)`.
theorem is_in_pair_of_eq_left[P: PartialOrder](a: P, b: P, x: P) {
    x = a implies is_in_pair(a, b, x)
}

/// `x = b` implies `is_in_pair(a, b, x)`.
theorem is_in_pair_of_eq_right[P: PartialOrder](a: P, b: P, x: P) {
    x = b implies is_in_pair(a, b, x)
}

/// `is_in_pair(a, a, x)` forces `x = a`.
theorem is_in_pair_self_imp_eq[P: PartialOrder](a: P, x: P) {
    is_in_pair(a, a, x) implies x = a
}

/// Membership in a pair is failure of both inequalities being strict.
theorem not_is_in_pair_iff[P: PartialOrder](a: P, b: P, x: P) {
    not is_in_pair(a, b, x) iff (x != a and x != b)
}

/// `is_in_pair(a, b)` and `pred_insert(a, is_singleton(b))` agree pointwise.
theorem is_in_pair_iff_insert_singleton[P: PartialOrder](a: P, b: P, x: P) {
    is_in_pair(a, b, x) iff pred_insert(a, is_singleton(b), x)
}

/// `is_in_pair(a, b)` and `pred_union(is_singleton(a), is_singleton(b))` agree pointwise.
theorem is_in_pair_iff_union_singletons[P: PartialOrder](a: P, b: P, x: P) {
    is_in_pair(a, b, x) iff pred_union(is_singleton(a), is_singleton(b), x)
}

/// `is_in_pair(a, a)` and `is_singleton(a)` agree pointwise.
theorem is_in_pair_self_iff_singleton[P: PartialOrder](a: P, x: P) {
    is_in_pair(a, a, x) iff is_singleton(a, x)
}

/// Any member of `is_in_pair(a, b)` is below the join.
theorem is_in_pair_lte_join[S: JoinSemilattice](a: S, b: S, x: S) {
    is_in_pair(a, b, x) implies x <= a.join(b)
} by {
    join_is_upper_bound(a, b)
}

/// Any member of `is_in_pair(a, b)` is above the meet.
theorem meet_lte_is_in_pair[S: MeetSemilattice](a: S, b: S, x: S) {
    is_in_pair(a, b, x) implies a.meet(b) <= x
} by {
    meet_is_lower_bound(a, b)
}

/// An infimum of `is_in_pair(a, b)` equals the meet.
theorem is_glb_is_in_pair_eq_meet[S: MeetSemilattice](a: S, b: S, c: S) {
    is_glb(is_in_pair(a, b), c) implies c = a.meet(b)
} by {
    meet_is_glb(a, b)
    is_glb_unique(is_in_pair(a, b), c, a.meet(b))
}

/// A supremum of `is_in_pair(a, b)` equals the join.
theorem is_lub_is_in_pair_eq_join[S: JoinSemilattice](a: S, b: S, c: S) {
    is_lub(is_in_pair(a, b), c) implies c = a.join(b)
} by {
    join_is_lub(a, b)
    is_lub_unique(is_in_pair(a, b), c, a.join(b))
}

/// An infimum of `is_in_pair(a, b)` is the meet, iff form.
theorem is_glb_is_in_pair_iff_eq_meet[S: MeetSemilattice](a: S, b: S, c: S) {
    is_glb(is_in_pair(a, b), c) iff c = a.meet(b)
} by {
    if is_glb(is_in_pair(a, b), c) {
        is_glb_is_in_pair_eq_meet(a, b, c)
    }
    if c = a.meet(b) {
        meet_is_glb(a, b)
    }
}

/// If `a <= b`, then `a` is the least element of `is_in_pair(a, b)`.
theorem is_least_is_in_pair_of_lte[P: PartialOrder](a: P, b: P) {
    a <= b implies is_least(is_in_pair(a, b), a)
} by {
    if a <= b {
        forall(x: P) {
            if is_in_pair(a, b, x) {
                if x = a {
                    a <= x
                } else {
                    x = b
                    a <= x
                }
            }
        }
        is_in_pair(a, b, a)
    }
}

/// If `a <= b`, then `b` is the greatest element of `is_in_pair(a, b)`.
theorem is_greatest_is_in_pair_of_lte[P: PartialOrder](a: P, b: P) {
    a <= b implies is_greatest(is_in_pair(a, b), b)
} by {
    if a <= b {
        forall(x: P) {
            if is_in_pair(a, b, x) {
                if x = a {
                    x <= b
                } else {
                    x = b
                    x <= b
                }
            }
        }
        is_in_pair(a, b, b)
    }
}

/// If `a <= b`, then `a` is the infimum of `is_in_pair(a, b)`.
theorem is_glb_is_in_pair_of_lte[P: PartialOrder](a: P, b: P) {
    a <= b implies is_glb(is_in_pair(a, b), a)
} by {
    is_least_is_in_pair_of_lte(a, b)
    is_least_is_glb(is_in_pair(a, b), a)
}

/// If `a <= b`, then `b` is the supremum of `is_in_pair(a, b)`.
theorem is_lub_is_in_pair_of_lte[P: PartialOrder](a: P, b: P) {
    a <= b implies is_lub(is_in_pair(a, b), b)
} by {
    is_greatest_is_in_pair_of_lte(a, b)
    is_greatest_is_lub(is_in_pair(a, b), b)
}

/// A supremum of `is_in_pair(a, b)` is the join, iff form.
theorem is_lub_is_in_pair_iff_eq_join[S: JoinSemilattice](a: S, b: S, c: S) {
    is_lub(is_in_pair(a, b), c) iff c = a.join(b)
} by {
    if is_lub(is_in_pair(a, b), c) {
        is_lub_is_in_pair_eq_join(a, b, c)
    }
    if c = a.join(b) {
        join_is_lub(a, b)
    }
}

/// If `b <= a`, then `b` is the least element of `is_in_pair(a, b)`.
theorem is_least_is_in_pair_of_lte_right[P: PartialOrder](a: P, b: P) {
    b <= a implies is_least(is_in_pair(a, b), b)
} by {
    if b <= a {
        is_least_is_in_pair_of_lte(b, a)
        forall(x: P) {
            is_in_pair_comm(b, a, x)
        }
        is_least_of_pred_eq(is_in_pair(b, a), is_in_pair(a, b), b)
    }
}

/// If `b <= a`, then `a` is the greatest element of `is_in_pair(a, b)`.
theorem is_greatest_is_in_pair_of_lte_right[P: PartialOrder](a: P, b: P) {
    b <= a implies is_greatest(is_in_pair(a, b), a)
} by {
    if b <= a {
        is_greatest_is_in_pair_of_lte(b, a)
        forall(x: P) {
            is_in_pair_comm(b, a, x)
        }
        is_greatest_of_pred_eq(is_in_pair(b, a), is_in_pair(a, b), a)
    }
}

/// If `b <= a`, then `b` is the infimum of `is_in_pair(a, b)`.
theorem is_glb_is_in_pair_of_lte_right[P: PartialOrder](a: P, b: P) {
    b <= a implies is_glb(is_in_pair(a, b), b)
} by {
    is_least_is_in_pair_of_lte_right(a, b)
    is_least_is_glb(is_in_pair(a, b), b)
}

/// If `b <= a`, then `a` is the supremum of `is_in_pair(a, b)`.
theorem is_lub_is_in_pair_of_lte_right[P: PartialOrder](a: P, b: P) {
    b <= a implies is_lub(is_in_pair(a, b), a)
} by {
    is_greatest_is_in_pair_of_lte_right(a, b)
    is_greatest_is_lub(is_in_pair(a, b), a)
}

/// `a` is the least element of `is_in_pair(a, b)` iff `a <= b`.
theorem is_least_is_in_pair_left_iff_lte[P: PartialOrder](a: P, b: P) {
    is_least(is_in_pair(a, b), a) iff a <= b
} by {
    if is_least(is_in_pair(a, b), a) {
        is_in_pair_right(a, b)
        is_lower_bound(is_in_pair(a, b), a)
        a <= b
    }
    if a <= b {
        is_least_is_in_pair_of_lte(a, b)
    }
}

/// `b` is the greatest element of `is_in_pair(a, b)` iff `a <= b`.
theorem is_greatest_is_in_pair_right_iff_lte[P: PartialOrder](a: P, b: P) {
    is_greatest(is_in_pair(a, b), b) iff a <= b
} by {
    if is_greatest(is_in_pair(a, b), b) {
        is_in_pair_left(a, b)
        is_upper_bound(is_in_pair(a, b), b)
        a <= b
    }
    if a <= b {
        is_greatest_is_in_pair_of_lte(a, b)
    }
}

/// `b` is the least element of `is_in_pair(a, b)` iff `b <= a`.
theorem is_least_is_in_pair_right_iff_lte[P: PartialOrder](a: P, b: P) {
    is_least(is_in_pair(a, b), b) iff b <= a
} by {
    if is_least(is_in_pair(a, b), b) {
        is_in_pair_left(a, b)
        is_lower_bound(is_in_pair(a, b), b)
        b <= a
    }
    if b <= a {
        is_least_is_in_pair_of_lte_right(a, b)
    }
}

/// `a` is the greatest element of `is_in_pair(a, b)` iff `b <= a`.
theorem is_greatest_is_in_pair_left_iff_lte[P: PartialOrder](a: P, b: P) {
    is_greatest(is_in_pair(a, b), a) iff b <= a
} by {
    if is_greatest(is_in_pair(a, b), a) {
        is_in_pair_right(a, b)
        is_upper_bound(is_in_pair(a, b), a)
        b <= a
    }
    if b <= a {
        is_greatest_is_in_pair_of_lte_right(a, b)
    }
}

/// The infimum of `is_in_pair(a, b)` is below the left endpoint.
theorem is_glb_is_in_pair_le_left[P: PartialOrder](a: P, b: P, x: P) {
    is_glb(is_in_pair(a, b), x) implies x <= a
} by {
    is_glb_is_lower_bound(is_in_pair(a, b), x)
    is_in_pair_left(a, b)
    lower_bound_le(is_in_pair(a, b), x, a)
}

/// The infimum of `is_in_pair(a, b)` is below the right endpoint.
theorem is_glb_is_in_pair_le_right[P: PartialOrder](a: P, b: P, x: P) {
    is_glb(is_in_pair(a, b), x) implies x <= b
} by {
    is_glb_is_lower_bound(is_in_pair(a, b), x)
    is_in_pair_right(a, b)
    lower_bound_le(is_in_pair(a, b), x, b)
}

/// The supremum of `is_in_pair(a, b)` is above the left endpoint.
theorem is_lub_is_in_pair_ge_left[P: PartialOrder](a: P, b: P, x: P) {
    is_lub(is_in_pair(a, b), x) implies a <= x
} by {
    is_lub_is_upper_bound(is_in_pair(a, b), x)
    is_in_pair_left(a, b)
    upper_bound_ge(is_in_pair(a, b), x, a)
}

/// The supremum of `is_in_pair(a, b)` is above the right endpoint.
theorem is_lub_is_in_pair_ge_right[P: PartialOrder](a: P, b: P, x: P) {
    is_lub(is_in_pair(a, b), x) implies b <= x
} by {
    is_lub_is_upper_bound(is_in_pair(a, b), x)
    is_in_pair_right(a, b)
    upper_bound_ge(is_in_pair(a, b), x, b)
}

/// `a` is the infimum of `is_in_pair(a, b)` iff `a <= b`.
theorem is_glb_is_in_pair_left_iff_lte[P: PartialOrder](a: P, b: P) {
    is_glb(is_in_pair(a, b), a) iff a <= b
} by {
    if is_glb(is_in_pair(a, b), a) {
        is_glb_is_lower_bound(is_in_pair(a, b), a)
        is_in_pair_right(a, b)
        lower_bound_le(is_in_pair(a, b), a, b)
    }
    if a <= b {
        is_glb_is_in_pair_of_lte(a, b)
    }
}

/// `b` is the supremum of `is_in_pair(a, b)` iff `a <= b`.
theorem is_lub_is_in_pair_right_iff_lte[P: PartialOrder](a: P, b: P) {
    is_lub(is_in_pair(a, b), b) iff a <= b
} by {
    if is_lub(is_in_pair(a, b), b) {
        is_lub_is_upper_bound(is_in_pair(a, b), b)
        is_in_pair_left(a, b)
        upper_bound_ge(is_in_pair(a, b), b, a)
    }
    if a <= b {
        is_lub_is_in_pair_of_lte(a, b)
    }
}

/// `b` is the infimum of `is_in_pair(a, b)` iff `b <= a`.
theorem is_glb_is_in_pair_right_iff_lte[P: PartialOrder](a: P, b: P) {
    is_glb(is_in_pair(a, b), b) iff b <= a
} by {
    if is_glb(is_in_pair(a, b), b) {
        is_glb_is_lower_bound(is_in_pair(a, b), b)
        is_in_pair_left(a, b)
        lower_bound_le(is_in_pair(a, b), b, a)
    }
    if b <= a {
        is_glb_is_in_pair_of_lte_right(a, b)
    }
}

/// `a` is the supremum of `is_in_pair(a, b)` iff `b <= a`.
theorem is_lub_is_in_pair_left_iff_lte[P: PartialOrder](a: P, b: P) {
    is_lub(is_in_pair(a, b), a) iff b <= a
} by {
    if is_lub(is_in_pair(a, b), a) {
        is_lub_is_upper_bound(is_in_pair(a, b), a)
        is_in_pair_right(a, b)
        upper_bound_ge(is_in_pair(a, b), a, b)
    }
    if b <= a {
        is_lub_is_in_pair_of_lte_right(a, b)
    }
}



/// `a` is the least element of `is_in_pair(a, a)`.
theorem is_least_is_in_pair_self[P: PartialOrder](a: P) {
    is_least(is_in_pair(a, a), a)
} by {
    is_least_is_in_pair_of_lte(a, a)
}

/// `a` is the greatest element of `is_in_pair(a, a)`.
theorem is_greatest_is_in_pair_self[P: PartialOrder](a: P) {
    is_greatest(is_in_pair(a, a), a)
} by {
    is_greatest_is_in_pair_of_lte(a, a)
}

/// `a` is the infimum of `is_in_pair(a, a)`.
theorem is_glb_is_in_pair_self[P: PartialOrder](a: P) {
    is_glb(is_in_pair(a, a), a)
} by {
    is_glb_is_in_pair_of_lte(a, a)
}

/// `a` is the supremum of `is_in_pair(a, a)`.
theorem is_lub_is_in_pair_self[P: PartialOrder](a: P) {
    is_lub(is_in_pair(a, a), a)
} by {
    is_lub_is_in_pair_of_lte(a, a)
}


/// A lower bound of `is_in_pair(a, a)` is exactly an element below `a`.
theorem is_lower_bound_is_in_pair_self_iff_le[P: PartialOrder](a: P, c: P) {
    is_lower_bound(is_in_pair(a, a), c) iff c <= a
} by {
    if is_lower_bound(is_in_pair(a, a), c) {
        is_in_pair_left(a, a)
        lower_bound_le(is_in_pair(a, a), c, a)
    }
    if c <= a {
        forall(x: P) {
            if is_in_pair(a, a, x) {
                is_in_pair_self_imp_eq(a, x)
                x = a
                c <= x
            }
        }
        is_lower_bound(is_in_pair(a, a), c)
    }
}

/// An upper bound of `is_in_pair(a, a)` is exactly an element above `a`.
theorem is_upper_bound_is_in_pair_self_iff_ge[P: PartialOrder](a: P, c: P) {
    is_upper_bound(is_in_pair(a, a), c) iff a <= c
} by {
    if is_upper_bound(is_in_pair(a, a), c) {
        is_in_pair_left(a, a)
        upper_bound_ge(is_in_pair(a, a), c, a)
    }
    if a <= c {
        forall(x: P) {
            if is_in_pair(a, a, x) {
                is_in_pair_self_imp_eq(a, x)
                x = a
                x <= c
            }
        }
        is_upper_bound(is_in_pair(a, a), c)
    }
}
