/// Optional values that can either contain a value of type T or be empty.
/// Useful for representing values that may or may not exist.
inductive Option[T] {
    /// `Option.none` represents the absence of a value.
    none

    /// `Option.some(value)` represents the presence of a value.
    some(T)
}

let some[T] = Option.some[T]

let none[T] = Option.none[T]

/// True if the optional value contains an element.
define option_is_some[T](opt: Option[T]) -> Bool {
    match opt {
        Option.none {
            false
        }
        Option.some(value) {
            true
        }
    }
}

/// True if the optional value contains no element.
define option_is_none[T](opt: Option[T]) -> Bool {
    match opt {
        Option.none {
            true
        }
        Option.some(value) {
            false
        }
    }
}

/// The image of an optional value under a function.
define option_map[T, U](opt: Option[T], f: T -> U) -> Option[U] {
    match opt {
        Option.none {
            Option.none[U]
        }
        Option.some(value) {
            Option.some(f(value))
        }
    }
}

/// The result of applying an optional-value function to an optional value.
define option_bind[T, U](opt: Option[T], f: T -> Option[U]) -> Option[U] {
    match opt {
        Option.none {
            Option.none[U]
        }
        Option.some(value) {
            f(value)
        }
    }
}

/// The contained value, or the given fallback when there is no contained value.
define option_get_or_else[T](opt: Option[T], fallback: T) -> T {
    match opt {
        Option.none {
            fallback
        }
        Option.some(value) {
            value
        }
    }
}

attributes Option[T] {
    /// True if the optional value contains an element.
    let is_some: Option[T] -> Bool = option_is_some

    /// True if the optional value contains no element.
    let is_none: Option[T] -> Bool = option_is_none

    /// The image of this optional value under a function.
    define map[U](self, f: T -> U) -> Option[U] {
        option_map(self, f)
    }

    /// The result of applying an optional-value function to this optional value.
    define bind[U](self, f: T -> Option[U]) -> Option[U] {
        option_bind(self, f)
    }

    /// The contained value, or the given fallback when there is no contained value.
    define get_or_else(self, fallback: T) -> T {
        option_get_or_else(self, fallback)
    }
}

/// `Option.some` is injective.
theorem some_injective[T](a: T, b: T) {
    Option.some(a) = Option.some(b) implies a = b
}

/// Mapping over `none` gives `none`.
theorem option_map_none[T, U](f: T -> U) {
    option_map(Option.none[T], f) = Option.none[U]
}

/// Mapping over `some` applies the function to the contained value.
theorem option_map_some[T, U](value: T, f: T -> U) {
    option_map(Option.some(value), f) = Option.some(f(value))
}

/// Binding over `none` gives `none`.
theorem option_bind_none[T, U](f: T -> Option[U]) {
    option_bind(Option.none[T], f) = Option.none[U]
}

/// Binding over `some` applies the function to the contained value.
theorem option_bind_some[T, U](value: T, f: T -> Option[U]) {
    option_bind(Option.some(value), f) = f(value)
}

/// The fallback for `none` is the given value.
theorem option_get_or_else_none[T](fallback: T) {
    option_get_or_else(Option.none[T], fallback) = fallback
}

/// The fallback for `some` is ignored.
theorem option_get_or_else_some[T](value: T, fallback: T) {
    option_get_or_else(Option.some(value), fallback) = value
}

/// constant[T, U](u) is a T -> U function that always returns u.
define constant[T, U](u: U, t: T) -> U {
    u
}
