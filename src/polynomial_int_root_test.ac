from nat import Nat, divides_lte
from int import Int, abs, div_imp_div_abs, abs_zero_imp_zero
from polynomial import Polynomial, coeff_tail, coeff_eval, polynomial_eval,
    polynomial_eval_bound, polynomial_eval_bound_eq_coeff_eval,
    polynomial_eval_eq_eval_bound_of_support_bounded, polynomial_support_bounded_by,
    polynomial_support_bounded_by_monotone, polynomial_support_bound_exists

numerals Int

/// True if an integer is a root of an integer polynomial.
///
/// Named rather than written as an equation at each use, so that the statements below read as
/// statements about roots.
define is_int_root(p: Polynomial[Int], r: Int) -> Bool {
    polynomial_eval(p, r) = Int.0
}

/// A root makes the evaluation vanish.
theorem is_int_root_apply(p: Polynomial[Int], r: Int) {
    is_int_root(p, r) implies polynomial_eval(p, r) = Int.0
} by {
    if is_int_root(p, r) {
        (is_int_root(p, r) = (polynomial_eval(p, r) = Int.0))
        polynomial_eval(p, r) = Int.0
    }
}

/// A vanishing evaluation is a root.
theorem is_int_root_intro(p: Polynomial[Int], r: Int) {
    polynomial_eval(p, r) = Int.0 implies is_int_root(p, r)
} by {
    if polynomial_eval(p, r) = Int.0 {
        (is_int_root(p, r) = (polynomial_eval(p, r) = Int.0))
        is_int_root(p, r)
    }
}

/// Every polynomial has a support bound above zero.
///
/// The evaluation split below needs a bound of the form `n + 1`, since it peels off the constant
/// coefficient; a bound of zero would make the evaluation vacuously zero.
theorem polynomial_support_bound_suc_exists(p: Polynomial[Int]) {
    exists(n: Nat) {
        polynomial_support_bounded_by(p, n.suc)
    }
} by {
    polynomial_support_bound_exists(p)
    exists(m: Nat) {
        polynomial_support_bounded_by(p, m)
    }
    let (k: Nat) satisfy {
        polynomial_support_bounded_by(p, k)
    }
    k <= k.suc
    polynomial_support_bounded_by_monotone(p, k, k.suc)
    polynomial_support_bounded_by(p, k.suc)
    exists(n: Nat) {
        polynomial_support_bounded_by(p, n.suc)
    }
}

/// The value at a point splits off the constant coefficient.
///
/// Horner's rule read once: the value is the constant coefficient plus the point times the value
/// of the polynomial with that coefficient dropped. This is the whole content of the root test.
theorem polynomial_int_eval_split(p: Polynomial[Int], r: Int, n: Nat) {
    polynomial_support_bounded_by(p, n.suc) implies polynomial_eval(p, r)
        = p.coeff(Nat.0) + r * coeff_eval(coeff_tail(p.coeff), r, n)
} by {
    if polynomial_support_bounded_by(p, n.suc) {
        polynomial_eval_eq_eval_bound_of_support_bounded(p, r, n.suc)
        polynomial_eval(p, r) = polynomial_eval_bound(p, r, n.suc)
        polynomial_eval_bound_eq_coeff_eval(p, r, n.suc)
        polynomial_eval_bound(p, r, n.suc) = coeff_eval(p.coeff, r, n.suc)
        (coeff_eval(p.coeff, r, n.suc)
            = p.coeff(Nat.0) + r * coeff_eval(coeff_tail(p.coeff), r, n))
        (polynomial_eval(p, r)
            = p.coeff(Nat.0) + r * coeff_eval(coeff_tail(p.coeff), r, n))
    }
}

/// An integer root divides the constant coefficient.
///
/// The integer root test. Every term above the constant one carries a factor of the point, so a
/// vanishing value makes the constant coefficient a multiple of it. This is what turns the
/// search for integer roots into a search through the divisors of one coefficient.
theorem int_root_divides_constant_coeff(p: Polynomial[Int], r: Int) {
    is_int_root(p, r) implies r.divides(p.coeff(Nat.0))
} by {
    if is_int_root(p, r) {
        is_int_root_apply(p, r)
        polynomial_eval(p, r) = Int.0
        polynomial_support_bound_suc_exists(p)
        exists(n: Nat) {
            polynomial_support_bounded_by(p, n.suc)
        }
        let (m: Nat) satisfy {
            polynomial_support_bounded_by(p, m.suc)
        }
        polynomial_int_eval_split(p, r, m)
        (polynomial_eval(p, r)
            = p.coeff(Nat.0) + r * coeff_eval(coeff_tail(p.coeff), r, m))
        (Int.0 = p.coeff(Nat.0) + r * coeff_eval(coeff_tail(p.coeff), r, m))
        (p.coeff(Nat.0) = -(r * coeff_eval(coeff_tail(p.coeff), r, m)))
        (-(r * coeff_eval(coeff_tail(p.coeff), r, m))
            = -coeff_eval(coeff_tail(p.coeff), r, m) * r)
        (-coeff_eval(coeff_tail(p.coeff), r, m) * r = p.coeff(Nat.0))
        exists(d: Int) {
            d * r = p.coeff(Nat.0)
        }
        (r.divides(p.coeff(Nat.0)) = exists(d: Int) { d * r = p.coeff(Nat.0) })
        r.divides(p.coeff(Nat.0))
    }
}

/// An integer failing to divide the constant coefficient is not a root.
///
/// The contrapositive, which is the form a search uses: a candidate that misses the divisibility
/// can be discarded without evaluating the polynomial.
theorem not_int_root_of_not_divides(p: Polynomial[Int], r: Int) {
    not r.divides(p.coeff(Nat.0)) implies not is_int_root(p, r)
} by {
    if not r.divides(p.coeff(Nat.0)) {
        if is_int_root(p, r) {
            int_root_divides_constant_coeff(p, r)
            r.divides(p.coeff(Nat.0))
            false
        }
        not is_int_root(p, r)
    }
}

/// Zero is a root exactly when the constant coefficient vanishes.
///
/// One direction is the root test at zero, which divides nothing but zero. The other is the
/// evaluation split, where dropping the constant coefficient leaves a term carrying a factor of
/// the point.
theorem int_root_zero_iff_constant_coeff_zero(p: Polynomial[Int]) {
    is_int_root(p, Int.0) = (p.coeff(Nat.0) = Int.0)
} by {
    if is_int_root(p, Int.0) {
        int_root_divides_constant_coeff(p, Int.0)
        Int.0.divides(p.coeff(Nat.0))
        (Int.0.divides(p.coeff(Nat.0)) = exists(d: Int) { d * Int.0 = p.coeff(Nat.0) })
        exists(d: Int) { d * Int.0 = p.coeff(Nat.0) }
        let (e: Int) satisfy {
            e * Int.0 = p.coeff(Nat.0)
        }
        (e * Int.0 = Int.0)
        p.coeff(Nat.0) = Int.0
    }
    if p.coeff(Nat.0) = Int.0 {
        polynomial_support_bound_suc_exists(p)
        exists(n: Nat) {
            polynomial_support_bounded_by(p, n.suc)
        }
        let (m: Nat) satisfy {
            polynomial_support_bounded_by(p, m.suc)
        }
        polynomial_int_eval_split(p, Int.0, m)
        (polynomial_eval(p, Int.0)
            = p.coeff(Nat.0) + Int.0 * coeff_eval(coeff_tail(p.coeff), Int.0, m))
        (Int.0 * coeff_eval(coeff_tail(p.coeff), Int.0, m) = Int.0)
        (Int.0 + Int.0 = Int.0)
        polynomial_eval(p, Int.0) = Int.0
        is_int_root_intro(p, Int.0)
        is_int_root(p, Int.0)
    }
    is_int_root(p, Int.0) = (p.coeff(Nat.0) = Int.0)
}

/// An integer root is no larger in size than the constant coefficient.
///
/// The bound that makes the divisor search finite: with a nonzero constant coefficient, the
/// candidates lie in a bounded range, so there are finitely many of them.
theorem int_root_abs_lte(p: Polynomial[Int], r: Int) {
    is_int_root(p, r) and p.coeff(Nat.0) != Int.0
        implies abs(r) <= abs(p.coeff(Nat.0))
} by {
    if is_int_root(p, r) and p.coeff(Nat.0) != Int.0 {
        int_root_divides_constant_coeff(p, r)
        r.divides(p.coeff(Nat.0))
        div_imp_div_abs(r, p.coeff(Nat.0))
        abs(r).divides(abs(p.coeff(Nat.0)))
        if abs(p.coeff(Nat.0)) = Nat.0 {
            abs_zero_imp_zero(p.coeff(Nat.0))
            p.coeff(Nat.0) = Int.0
            false
        }
        abs(p.coeff(Nat.0)) != Nat.0
        divides_lte(abs(r), abs(p.coeff(Nat.0)))
        (abs(p.coeff(Nat.0)) = Nat.0 or abs(r) <= abs(p.coeff(Nat.0)))
        abs(r) <= abs(p.coeff(Nat.0))
    }
}
