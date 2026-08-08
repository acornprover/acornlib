from comm_ring import CommRing
from algebra.ring.ring_hom import RingHom
from algebra.ring.ideal import PrimeIdeal, ideal_ext, ideal_preimage, ideal_preimage_contains_eq,
    prime_ideal_as_ideal_contains_eq, prime_ideal_preimage,
    prime_ideal_preimage_contains_eq

/// The underlying ideal of a prime-ideal preimage has the same members as the ordinary ideal preimage.
theorem prime_ideal_preimage_as_ideal_contains_eq[R: CommRing, S: CommRing](
    f: RingHom[R, S], p: PrimeIdeal[S], x: R
) {
    prime_ideal_preimage(f, p).as_ideal.contains(x) = ideal_preimage(f, p.as_ideal).contains(x)
} by {
    prime_ideal_as_ideal_contains_eq(prime_ideal_preimage(f, p), x)
    prime_ideal_preimage(f, p).as_ideal.contains(x) = prime_ideal_preimage(f, p).contains(x)
    prime_ideal_preimage_contains_eq(f, p, x)
    prime_ideal_preimage(f, p).contains(x) = p.contains(f.hom(x))
    ideal_preimage_contains_eq(f, p.as_ideal, x)
    ideal_preimage(f, p.as_ideal).contains(x) = p.as_ideal.contains(f.hom(x))
    prime_ideal_as_ideal_contains_eq(p, f.hom(x))
    p.as_ideal.contains(f.hom(x)) = p.contains(f.hom(x))
    prime_ideal_preimage(f, p).as_ideal.contains(x) = ideal_preimage(f, p.as_ideal).contains(x)
}

/// Forgetting prime structure commutes with taking preimages under a ring homomorphism.
theorem prime_ideal_preimage_as_ideal_eq[R: CommRing, S: CommRing](
    f: RingHom[R, S], p: PrimeIdeal[S]
) {
    prime_ideal_preimage(f, p).as_ideal = ideal_preimage(f, p.as_ideal)
} by {
    forall(x: R) {
        prime_ideal_preimage_as_ideal_contains_eq(f, p, x)
        prime_ideal_preimage(f, p).as_ideal.contains(x) = ideal_preimage(f, p.as_ideal).contains(x)
    }
    ideal_ext(prime_ideal_preimage(f, p).as_ideal, ideal_preimage(f, p.as_ideal))
}
