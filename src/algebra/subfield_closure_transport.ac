/// Transport of generated subfields along field homomorphisms.

from algebra.field.field import Field
from algebra.field.field_hom import FieldHom
from data.basic.set import Set, set_image, set_image_contains_witness, maps_into_set_image,
    set_preimage, set_preimage_contains_eq
from algebra.subfield import subfield_le, subfield_le_intro, subfield_le_trans,
    subfield_le_antisymm
from algebra.subfield_closure import set_subset_subfield, set_subset_subfield_intro,
    set_subset_subfield_contains, subfield_closure,
    subfield_closure_contains_generator, subfield_closure_minimal
from algebra.field.field_hom_image import field_hom_image, field_hom_image_contains,
    field_hom_image_contains_eq, subfield_image, subfield_image_contains_eq,
    subfield_image_contains_hom, subfield_image_le, subfield_preimage,
    subfield_preimage_contains_eq, subfield_image_preimage_le,
    subfield_preimage_image_eq

/// The image of the generators lies in the image of the generated subfield.
theorem set_image_subset_subfield_image_closure[F: Field, E: Field](
    phi: FieldHom[F, E], a: Set[F]
) {
    set_subset_subfield(set_image(a, phi.hom), subfield_image(phi, subfield_closure(a)))
} by {
    forall(y: E) {
        if set_image(a, phi.hom).contains(y) {
            set_image_contains_witness(a, phi.hom, y)
            let x: F satisfy { a.contains(x) and y = phi.hom(x) }
            subfield_closure_contains_generator(a, x)
            subfield_closure(a).contains(x)
            subfield_image_contains_hom(phi, subfield_closure(a), x)
            subfield_image(phi, subfield_closure(a)).contains(phi.hom(x))
            subfield_image(phi, subfield_closure(a)).contains(y)
        }
    }
    set_subset_subfield_intro(set_image(a, phi.hom), subfield_image(phi, subfield_closure(a)))
}

/// The image of the generated subfield is contained in the generated image subfield.
theorem subfield_image_closure_le_closure_image[F: Field, E: Field](
    phi: FieldHom[F, E], a: Set[F]
) {
    subfield_le(subfield_image(phi, subfield_closure(a)), subfield_closure(set_image(a, phi.hom)))
} by {
    let target = subfield_closure(set_image(a, phi.hom))
    let pre = subfield_preimage(phi, target)
    forall(x: F) {
        if a.contains(x) {
            maps_into_set_image(a, phi.hom, x)
            set_image(a, phi.hom).contains(phi.hom(x))
            subfield_closure_contains_generator(set_image(a, phi.hom), phi.hom(x))
            target.contains(phi.hom(x))
            subfield_preimage_contains_eq(phi, target, x)
            pre.contains(x)
        }
    }
    set_subset_subfield_intro(a, pre)
    set_subset_subfield(a, pre)
    subfield_closure_minimal(a, pre)
    subfield_le(subfield_closure(a), pre)
    forall(y: E) {
        if subfield_image(phi, subfield_closure(a)).contains(y) {
            subfield_image_contains_eq(phi, subfield_closure(a), y)
            let x: F satisfy { subfield_closure(a).contains(x) and phi.hom(x) = y }
            pre.contains(x)
            subfield_preimage_contains_eq(phi, target, x)
            target.contains(phi.hom(x))
            target.contains(y)
        }
    }
    subfield_le_intro(subfield_image(phi, subfield_closure(a)), target)
}

/// The generated subfield commutes with forward image under a field homomorphism.
theorem subfield_image_closure_eq_closure_image[F: Field, E: Field](
    phi: FieldHom[F, E], a: Set[F]
) {
    subfield_image(phi, subfield_closure(a)) = subfield_closure(set_image(a, phi.hom))
} by {
    set_image_subset_subfield_image_closure(phi, a)
    subfield_closure_minimal(set_image(a, phi.hom), subfield_image(phi, subfield_closure(a)))
    subfield_le(subfield_closure(set_image(a, phi.hom)), subfield_image(phi, subfield_closure(a)))
    subfield_image_closure_le_closure_image(phi, a)
    subfield_le(subfield_image(phi, subfield_closure(a)), subfield_closure(set_image(a, phi.hom)))
    subfield_le_antisymm(subfield_image(phi, subfield_closure(a)), subfield_closure(set_image(a, phi.hom)))
}

/// Pulling back the generated image subfield recovers the original generated subfield.
theorem subfield_preimage_closure_image_eq_closure[F: Field, E: Field](
    phi: FieldHom[F, E], a: Set[F]
) {
    subfield_preimage(phi, subfield_closure(set_image(a, phi.hom))) = subfield_closure(a)
} by {
    subfield_image_closure_eq_closure_image(phi, a)
    subfield_closure(set_image(a, phi.hom)) = subfield_image(phi, subfield_closure(a))
    subfield_preimage(phi, subfield_closure(set_image(a, phi.hom))) =
        subfield_preimage(phi, subfield_image(phi, subfield_closure(a)))
    subfield_preimage_image_eq(phi, subfield_closure(a))
}

/// The subfield generated by the set-theoretic preimage is contained in the preimage of the generated subfield.
theorem subfield_closure_preimage_le_preimage_closure[F: Field, E: Field](
    phi: FieldHom[F, E], a: Set[E]
) {
    subfield_le(subfield_closure(set_preimage(phi.hom, a)), subfield_preimage(phi, subfield_closure(a)))
} by {
    let target = subfield_preimage(phi, subfield_closure(a))
    forall(x: F) {
        if set_preimage(phi.hom, a).contains(x) {
            set_preimage_contains_eq(phi.hom, a, x)
            a.contains(phi.hom(x))
            subfield_closure_contains_generator(a, phi.hom(x))
            subfield_closure(a).contains(phi.hom(x))
            subfield_preimage_contains_eq(phi, subfield_closure(a), x)
            target.contains(x)
        }
    }
    set_subset_subfield_intro(set_preimage(phi.hom, a), target)
    set_subset_subfield(set_preimage(phi.hom, a), target)
    subfield_closure_minimal(set_preimage(phi.hom, a), target)
}

/// The image of the generated preimage subfield is contained in the generated codomain subfield.
theorem subfield_image_closure_preimage_le_closure[F: Field, E: Field](
    phi: FieldHom[F, E], a: Set[E]
) {
    subfield_le(subfield_image(phi, subfield_closure(set_preimage(phi.hom, a))), subfield_closure(a))
} by {
    subfield_closure_preimage_le_preimage_closure(phi, a)
    subfield_le(subfield_closure(set_preimage(phi.hom, a)), subfield_preimage(phi, subfield_closure(a)))
    subfield_image_le(phi, subfield_closure(set_preimage(phi.hom, a)), subfield_preimage(phi, subfield_closure(a)))
    subfield_le(subfield_image(phi, subfield_closure(set_preimage(phi.hom, a))),
        subfield_image(phi, subfield_preimage(phi, subfield_closure(a))))
    subfield_image_preimage_le(phi, subfield_closure(a))
    subfield_le(subfield_image(phi, subfield_preimage(phi, subfield_closure(a))), subfield_closure(a))
    subfield_le_trans(subfield_image(phi, subfield_closure(set_preimage(phi.hom, a))),
        subfield_image(phi, subfield_preimage(phi, subfield_closure(a))), subfield_closure(a))
}

/// If the generators lie in the image of phi, closure of preimage maps onto their generated subfield.
theorem subfield_image_closure_preimage_eq_closure_of_set_subset_image[F: Field, E: Field](
    phi: FieldHom[F, E], a: Set[E]
) {
    set_subset_subfield(a, field_hom_image(phi)) implies
    subfield_image(phi, subfield_closure(set_preimage(phi.hom, a))) = subfield_closure(a)
} by {
    let lhs = subfield_image(phi, subfield_closure(set_preimage(phi.hom, a)))
    let rhs = subfield_closure(a)
    if set_subset_subfield(a, field_hom_image(phi)) {
        subfield_image_closure_preimage_le_closure(phi, a)
        subfield_le(lhs, rhs)
        forall(y: E) {
            if a.contains(y) {
                set_subset_subfield_contains(a, field_hom_image(phi), y)
                field_hom_image(phi).contains(y)
                field_hom_image_contains_eq(phi)
                field_hom_image_contains(phi, y)
                let x: F satisfy { phi.hom(x) = y }
                a.contains(phi.hom(x))
                set_preimage_contains_eq(phi.hom, a, x)
                set_preimage(phi.hom, a).contains(x)
                subfield_closure_contains_generator(set_preimage(phi.hom, a), x)
                subfield_closure(set_preimage(phi.hom, a)).contains(x)
                subfield_image_contains_hom(phi, subfield_closure(set_preimage(phi.hom, a)), x)
                lhs.contains(phi.hom(x))
                lhs.contains(y)
            }
        }
        set_subset_subfield_intro(a, lhs)
        set_subset_subfield(a, lhs)
        subfield_closure_minimal(a, lhs)
        subfield_le(rhs, lhs)
        subfield_le_antisymm(lhs, rhs)
        lhs = rhs
        subfield_image(phi, subfield_closure(set_preimage(phi.hom, a))) = subfield_closure(a)
    }
}
