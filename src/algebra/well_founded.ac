/// Well-founded relations: every nonempty predicate admits an `r`-minimal element.

from nat import Nat, strong_induction, true_below, lt_suc, lt_trans, lt_or_lte
from pair import Pair, pair_eta, pair_new_first
from data.basic.functions import function_extensionality, binary_function_extensionality, is_bijection_fn,
    is_injective_fn, is_surjective_fn, bijection_fn_is_injective, bijection_fn_is_surjective,
    Bijection, bijection_map_is_bijection
from data.basic.relation_basic import is_irreflexive, relation_intersection, relation_intersection_left_subset,
    relation_intersection_right_subset, relation_subset, relation_subset_step, relation_converse,
    relation_converse_monotone
from data.basic.relation_transport import relation_pullback, relation_pushforward,
    relation_pullback_converse, relation_pushforward_converse
from data.basic.witness import choose_or_default, choose_or_default_spec, choose_or_default_unique_eq,
    choose_or_default_unique_default_eq, exists_unique, exists_unique_of_exists_and_pairwise

/// True if every nonempty predicate on `T` has an element that is `r`-minimal,
/// i.e. has no `r`-predecessor inside the predicate.
define is_well_founded[T](r: (T, T) -> Bool) -> Bool {
    forall(p: T -> Bool) {
        (exists(x: T) { p(x) }) implies exists(m: T) {
            p(m) and forall(y: T) {
                r(y, m) implies not p(y)
            }
        }
    }
}

/// True if a sequence descends by one `r`-step at every successor stage.
define is_descending_chain[T](r: (T, T) -> Bool, c: Nat -> T) -> Bool {
    forall(n: Nat) {
        r(c(n.suc), c(n))
    }
}

/// True if there is an infinite sequence descending by one `r`-step at each successor.
define has_descending_chain[T](r: (T, T) -> Bool) -> Bool {
    exists(c: Nat -> T) {
        is_descending_chain(r, c)
    }
}

/// True if there is no infinite sequence descending by one `r`-step at each successor.
define has_no_descending_chain[T](r: (T, T) -> Bool) -> Bool {
    not has_descending_chain(r)
}

/// True if every nonempty predicate has an `r`-maximal element.
define is_noetherian_relation[T](r: (T, T) -> Bool) -> Bool {
    is_well_founded(relation_converse(r))
}

/// True if a predicate contains an element whenever it contains all its `r`-successors.
define is_successor_closed_predicate[T](r: (T, T) -> Bool, p: T -> Bool) -> Bool {
    forall(x: T) {
        (forall(y: T) { r(x, y) implies p(y) }) implies p(x)
    }
}

/// A successor-closed predicate contains an element once it contains all successors of that element.
theorem successor_closed_predicate_at[T](r: (T, T) -> Bool, p: T -> Bool, x: T) {
    is_successor_closed_predicate(r, p) and
    forall(y: T) {
        r(x, y) implies p(y)
    }
    implies
    p(x)
} by {
    if is_successor_closed_predicate(r, p) and
        forall(y: T) {
            r(x, y) implies p(y)
        } {
        is_successor_closed_predicate(r, p) = forall(a: T) {
            (forall(b: T) { r(a, b) implies p(b) }) implies p(a)
        }
        p(x)
    }
}

/// True if a sequence ascends by one `r`-step at every successor stage.
define is_ascending_chain[T](r: (T, T) -> Bool, c: Nat -> T) -> Bool {
    forall(n: Nat) {
        r(c(n), c(n.suc))
    }
}

/// True if there is an infinite sequence ascending by one `r`-step at each successor.
define has_ascending_chain[T](r: (T, T) -> Bool) -> Bool {
    exists(c: Nat -> T) {
        is_ascending_chain(r, c)
    }
}

/// True if there is no infinite sequence ascending by one `r`-step at each successor.
define has_no_ascending_chain[T](r: (T, T) -> Bool) -> Bool {
    not has_ascending_chain(r)
}

/// An ascending chain for a relation is a descending chain for its converse.
theorem ascending_chain_iff_descending_chain_converse[T](r: (T, T) -> Bool, c: Nat -> T) {
    is_ascending_chain(r, c) = is_descending_chain(relation_converse(r), c)
} by {
    if is_ascending_chain(r, c) {
        forall(n: Nat) {
            r(c(n), c(n.suc))
            relation_converse(r, c(n.suc), c(n))
        }
        is_descending_chain(relation_converse(r), c)
    }
    if is_descending_chain(relation_converse(r), c) {
        forall(n: Nat) {
            relation_converse(r, c(n.suc), c(n))
            relation_converse(r, c(n.suc), c(n)) = r(c(n), c(n.suc))
            r(c(n), c(n.suc))
        }
        is_ascending_chain(r, c)
    }
    is_ascending_chain(r, c) = is_descending_chain(relation_converse(r), c)
}

/// Existence of an ascending chain is existence of a descending chain for the converse.
theorem has_ascending_chain_iff_has_descending_chain_converse[T](r: (T, T) -> Bool) {
    has_ascending_chain(r) = has_descending_chain(relation_converse(r))
} by {
    if has_ascending_chain(r) {
        let c: Nat -> T satisfy {
            is_ascending_chain(r, c)
        }
        ascending_chain_iff_descending_chain_converse(r, c)
        is_descending_chain(relation_converse(r), c)
        exists(d: Nat -> T) {
            d = c and is_descending_chain(relation_converse(r), d)
        }
        has_descending_chain(relation_converse(r))
    }
    if has_descending_chain(relation_converse(r)) {
        let c: Nat -> T satisfy {
            is_descending_chain(relation_converse(r), c)
        }
        ascending_chain_iff_descending_chain_converse(r, c)
        is_ascending_chain(r, c)
        exists(d: Nat -> T) {
            d = c and is_ascending_chain(r, d)
        }
        has_ascending_chain(r)
    }
    has_ascending_chain(r) = has_descending_chain(relation_converse(r))
}

/// The image sequence of a descending chain under a map.
define descending_chain_map[A, B](f: A -> B, c: Nat -> A, n: Nat) -> B {
    f(c(n))
}

/// The first-coordinate sequence of a pair-valued chain.
define descending_chain_first[A, B](c: Nat -> Pair[A, B], n: Nat) -> A {
    c(n).first
}

/// The second-coordinate sequence of a pair-valued chain.
define descending_chain_second[A, B](c: Nat -> Pair[A, B], n: Nat) -> B {
    c(n).second
}

/// The strict order relation on natural numbers, written as a relation.
define nat_lt_relation(x: Nat, y: Nat) -> Bool {
    x < y
}

/// True if `m` is the least natural number satisfying `p`.
define is_least_nat_witness(p: Nat -> Bool, m: Nat) -> Bool {
    p(m) and forall(y: Nat) {
        y < m implies not p(y)
    }
}

/// True if `m` is the least natural number where `p` fails.
define is_least_nat_counterexample(p: Nat -> Bool, m: Nat) -> Bool {
    not p(m) and forall(y: Nat) {
        y < m implies p(y)
    }
}

/// True if two functions agree on every `r`-predecessor of an element.
define functions_agree_on_predecessors[T, U](
    r: (T, T) -> Bool,
    x: T,
    f: T -> U,
    g: T -> U
) -> Bool {
    forall(y: T) {
        r(y, x) implies f(y) = g(y)
    }
}

/// True if a recursive step depends only on the values at `r`-predecessors.
define is_predecessor_extensional_recursive_step[T, U](
    r: (T, T) -> Bool,
    step: (T, T -> U) -> U
) -> Bool {
    forall(x: T, f: T -> U, g: T -> U) {
        functions_agree_on_predecessors(r, x, f, g) implies step(x, f) = step(x, g)
    }
}

/// True if a function satisfies a recursive equation over a relation.
define is_well_founded_recursive_solution[T, U](
    r: (T, T) -> Bool,
    step: (T, T -> U) -> U,
    f: T -> U
) -> Bool {
    forall(x: T) {
        f(x) = step(x, f)
    }
}

/// The predicate of being a solution to a well-founded recursive equation.
define well_founded_recursive_solution_predicate[T, U](
    r: (T, T) -> Bool,
    step: (T, T -> U) -> U,
    f: T -> U
) -> Bool {
    is_well_founded_recursive_solution(r, step, f)
}

/// A default function used when choosing a recursive solution without a witness.
define well_founded_recursive_default_function[T, U](default: U, x: T) -> U {
    default
}

/// The chosen recursive solution when one exists, and otherwise the constant default function.
define choose_well_founded_recursive_solution[T, U](
    r: (T, T) -> Bool,
    step: (T, T -> U) -> U,
    default: U
) -> T -> U {
    choose_or_default(
        well_founded_recursive_solution_predicate(r, step),
        well_founded_recursive_default_function[T, U](default)
    )
}

/// The recursive step determined by values assigned directly to each element.
define well_founded_nonrecursive_step[T, U](value: T -> U, x: T, f: T -> U) -> U {
    value(x)
}

/// The direct function determined by values assigned to each element.
define well_founded_nonrecursive_function[T, U](value: T -> U, x: T) -> U {
    value(x)
}

/// True if a function satisfies a recursive equation over the strict natural order.
define is_nat_recursive_solution[U](step: (Nat, Nat -> U) -> U, f: Nat -> U) -> Bool {
    is_well_founded_recursive_solution(nat_lt_relation, step, f)
}

/// The predicate of being a solution to a recursive equation over the strict natural order.
define nat_recursive_solution_predicate[U](step: (Nat, Nat -> U) -> U, f: Nat -> U) -> Bool {
    is_nat_recursive_solution(step, f)
}

/// The chosen recursive solution over the strict natural order when one exists.
define choose_nat_recursive_solution[U](step: (Nat, Nat -> U) -> U, default: U) -> Nat -> U {
    choose_well_founded_recursive_solution(nat_lt_relation, step, default)
}

/// The strict-natural-order recursive step determined directly by each natural number.
define nat_nonrecursive_step[U](value: Nat -> U, n: Nat, f: Nat -> U) -> U {
    value(n)
}

/// The direct function determined by values assigned to each natural number.
define nat_nonrecursive_function[U](value: Nat -> U, n: Nat) -> U {
    value(n)
}

/// The function defined by primitive recursion on natural numbers.
define nat_primitive_recursive_function[U](base: U, next: (Nat, U) -> U, n: Nat) -> U {
    match n {
        Nat.zero {
            base
        }
        Nat.suc(pred) {
            next(pred, nat_primitive_recursive_function(base, next, pred))
        }
    }
}

/// The recursive step determined by primitive recursion data on natural numbers.
define nat_primitive_recursive_step[U](base: U, next: (Nat, U) -> U, n: Nat, f: Nat -> U) -> U {
    match n {
        Nat.zero {
            base
        }
        Nat.suc(pred) {
            next(pred, f(pred))
        }
    }
}

/// The chosen primitive-recursive function determined by primitive recursion data.
define chosen_nat_primitive_recursive_function[U](base: U, next: (Nat, U) -> U, default: U, n: Nat) -> U {
    choose_nat_recursive_solution(nat_primitive_recursive_step(base, next), default)(n)
}

/// The state update for two-step recursion.
define nat_two_step_recursive_state_next[U](next: (Nat, U, U) -> U, n: Nat, state: Pair[U, U]) -> Pair[U, U] {
    Pair.new(state.second, next(n, state.first, state.second))
}

/// The state pair at a natural number for two-step recursion.
define nat_two_step_recursive_state[U](initial: Pair[U, U], next: (Nat, U, U) -> U, n: Nat) -> Pair[U, U] {
    nat_primitive_recursive_function(initial, nat_two_step_recursive_state_next(next), n)
}

/// The function determined by two-step recursion on natural numbers.
define nat_two_step_recursive_function[U](initial: Pair[U, U], next: (Nat, U, U) -> U, n: Nat) -> U {
    nat_two_step_recursive_state(initial, next, n).first
}

/// The recursive step determined by two-step recursion data on natural numbers.
define nat_two_step_recursive_step[U](initial: Pair[U, U], next: (Nat, U, U) -> U, n: Nat, f: Nat -> U) -> U {
    match n {
        Nat.zero {
            initial.first
        }
        Nat.suc(pred) {
            match pred {
                Nat.zero {
                    initial.second
                }
                Nat.suc(prev) {
                    next(prev, f(prev), f(pred))
                }
            }
        }
    }
}

/// The chosen two-step recursive function determined by two-step recursion data.
define chosen_nat_two_step_recursive_function[U](
    initial: Pair[U, U],
    next: (Nat, U, U) -> U,
    default: U,
    n: Nat
) -> U {
    choose_nat_recursive_solution(nat_two_step_recursive_step(initial, next), default)(n)
}

/// The lexicographic relation on pairs.
define lex_relation[A, B](r: (A, A) -> Bool, s: (B, B) -> Bool, p: Pair[A, B], q: Pair[A, B]) -> Bool {
    r(p.first, q.first) or (p.first = q.first and s(p.second, q.second))
}

/// The first-coordinate product relation on pairs.
define product_first_relation[A, B](r: (A, A) -> Bool, p: Pair[A, B], q: Pair[A, B]) -> Bool {
    r(p.first, q.first)
}

/// The second-coordinate product relation on pairs.
define product_second_relation[A, B](s: (B, B) -> Bool, p: Pair[A, B], q: Pair[A, B]) -> Bool {
    s(p.second, q.second)
}

/// The lexicographic relation contains every first-coordinate product step.
theorem product_first_relation_subset_lex_relation[A, B](r: (A, A) -> Bool, s: (B, B) -> Bool) {
    relation_subset(product_first_relation[A, B](r), lex_relation(r, s))
} by {
    forall(p: Pair[A, B], q: Pair[A, B]) {
        if product_first_relation(r, p, q) {
            r(p.first, q.first)
            lex_relation(r, s, p, q)
        }
    }
}

/// The lexicographic relation contains every second-coordinate step within a fixed fiber.
theorem product_second_relation_subset_lex_relation_same_first[A, B](
    r: (A, A) -> Bool,
    s: (B, B) -> Bool,
    p: Pair[A, B],
    q: Pair[A, B]
) {
    product_second_relation[A, B](s, p, q) and p.first = q.first implies lex_relation(r, s, p, q)
} by {
    if product_second_relation(s, p, q) and p.first = q.first {
        s(p.second, q.second)
        p.first = q.first and s(p.second, q.second)
        lex_relation(r, s, p, q)
    }
}


/// A well-founded relation is irreflexive.
theorem well_founded_is_irreflexive[T](r: (T, T) -> Bool) {
    is_well_founded(r) implies is_irreflexive(r)
} by {
    if is_well_founded(r) {
        is_well_founded(r) = forall(p: T -> Bool) {
            (exists(x: T) { p(x) }) implies exists(m: T) {
                p(m) and forall(y: T) {
                    r(y, m) implies not p(y)
                }
            }
        }
        forall(x: T) {
            let p: T -> Bool = function(z: T) { z = x }
            p(x) = (x = x)
            p(x)
            exists(z: T) { p(z) }
            exists(m: T) {
                p(m) and forall(y: T) {
                    r(y, m) implies not p(y)
                }
            }
            let m: T satisfy {
                p(m) and forall(y: T) {
                    r(y, m) implies not p(y)
                }
            }
            p(m)
            m = x
            forall(y: T) {
                r(y, m) implies not p(y)
            }
            if r(x, x) {
                r(x, m)
                not p(x)
                false
            }
            not r(x, x)
        }
        is_irreflexive(r)
    }
}

/// The pullback of a well-founded relation along any function is well-founded.
theorem relation_pullback_is_well_founded[A, B](f: A -> B, r: (B, B) -> Bool) {
    is_well_founded(r) implies is_well_founded(relation_pullback(f, r))
} by {
    if is_well_founded(r) {
        is_well_founded(r) = forall(p: B -> Bool) {
            (exists(x: B) { p(x) }) implies exists(m: B) {
                p(m) and forall(y: B) {
                    r(y, m) implies not p(y)
                }
            }
        }
        forall(p: A -> Bool) {
            if exists(x: A) { p(x) } {
                let q: B -> Bool = function(b: B) {
                    exists(a: A) { f(a) = b and p(a) }
                }
                let x0: A satisfy { p(x0) }
                q(f(x0))
                exists(b: B) { q(b) }
                exists(mb: B) {
                    q(mb) and forall(yb: B) {
                        r(yb, mb) implies not q(yb)
                    }
                }
                let mb: B satisfy {
                    q(mb) and forall(yb: B) {
                        r(yb, mb) implies not q(yb)
                    }
                }
                q(mb)
                let ma: A satisfy { f(ma) = mb and p(ma) }
                forall(ya: A) {
                    if relation_pullback(f, r, ya, ma) {
                        r(f(ya), f(ma))
                        r(f(ya), mb)
                        not q(f(ya))
                        if p(ya) {
                            f(ya) = f(ya)
                            q(f(ya))
                            false
                        }
                        not p(ya)
                    }
                }
                exists(m: A) {
                    p(m) and forall(y: A) {
                        relation_pullback(f, r, y, m) implies not p(y)
                    }
                }
            }
        }
        is_well_founded(relation_pullback(f, r))
    }
}

/// The pushforward of a well-founded relation along a bijection is well-founded.
theorem relation_pushforward_is_well_founded_of_bijection[A, B](f: A -> B, r: (A, A) -> Bool) {
    is_bijection_fn(f) and is_well_founded(r) implies is_well_founded(relation_pushforward(f, r))
} by {
    if is_bijection_fn(f) and is_well_founded(r) {
        is_well_founded(r) = forall(p: A -> Bool) {
            (exists(x: A) { p(x) }) implies exists(m: A) {
                p(m) and forall(y: A) {
                    r(y, m) implies not p(y)
                }
            }
        }
        bijection_fn_is_injective(f)
        is_injective_fn(f)
        bijection_fn_is_surjective(f)
        is_surjective_fn(f)
        is_surjective_fn(f) = forall(y: B) {
            exists(x: A) {
                f(x) = y
            }
        }
        forall(q: B -> Bool) {
            if exists(y: B) { q(y) } {
                let p: A -> Bool = function(x: A) {
                    q(f(x))
                }
                let y0: B satisfy { q(y0) }
                exists(x0: A) {
                    f(x0) = y0
                }
                let x0: A satisfy {
                    f(x0) = y0
                }
                q(f(x0))
                p(x0)
                exists(x: A) { p(x) }
                exists(m: A) {
                    p(m) and forall(y: A) {
                        r(y, m) implies not p(y)
                    }
                }
                let m: A satisfy {
                    p(m) and forall(y: A) {
                        r(y, m) implies not p(y)
                    }
                }
                p(m)
                q(f(m))
                let n: B = f(m)
                forall(y: B) {
                    if relation_pushforward(f, r, y, n) {
                        relation_pushforward(f, r, y, n) = exists(a: A, b: A) {
                            f(a) = y and f(b) = n and r(a, b)
                        }
                        let (a: A, b: A) satisfy {
                            f(a) = y and f(b) = n and r(a, b)
                        }
                        n = f(m)
                        f(b) = f(m)
                        b = m
                        r(a, m)
                        forall(z: A) {
                            r(z, m) implies not p(z)
                        }
                        not p(a)
                        if q(y) {
                            f(a) = y
                            q(f(a))
                            p(a)
                            false
                        }
                        not q(y)
                    }
                }
                q(n) and forall(y: B) {
                    relation_pushforward(f, r, y, n) implies not q(y)
                }
                exists(minimal: B) {
                    q(minimal) and forall(y: B) {
                        relation_pushforward(f, r, y, minimal) implies not q(y)
                    }
                }
            }
        }
        is_well_founded(relation_pushforward(f, r)) = forall(q: B -> Bool) {
            (exists(y: B) { q(y) }) implies exists(minimal: B) {
                q(minimal) and forall(y: B) {
                    relation_pushforward(f, r, y, minimal) implies not q(y)
                }
            }
        }
        is_well_founded(relation_pushforward(f, r))
    }
}

/// The pushforward of a well-founded relation along an injection is well-founded.
theorem relation_pushforward_is_well_founded_of_injection[A, B](f: A -> B, r: (A, A) -> Bool) {
    is_injective_fn(f) and is_well_founded(r) implies is_well_founded(relation_pushforward(f, r))
} by {
    if is_injective_fn(f) and is_well_founded(r) {
        is_well_founded(r) = forall(p: A -> Bool) {
            (exists(x: A) { p(x) }) implies exists(m: A) {
                p(m) and forall(y: A) {
                    r(y, m) implies not p(y)
                }
            }
        }
        is_injective_fn(f) = forall(x1: A, x2: A) {
            f(x1) = f(x2) implies x1 = x2
        }
        forall(q: B -> Bool) {
            if exists(y: B) { q(y) } {
                let p: A -> Bool = function(x: A) {
                    q(f(x))
                }
                let y0: B satisfy { q(y0) }
                if exists(x: A) { p(x) } {
                    exists(m: A) {
                        p(m) and forall(y: A) {
                            r(y, m) implies not p(y)
                        }
                    }
                    let m: A satisfy {
                        p(m) and forall(y: A) {
                            r(y, m) implies not p(y)
                        }
                    }
                    p(m)
                    q(f(m))
                    let n: B = f(m)
                    forall(y: B) {
                        if relation_pushforward(f, r, y, n) {
                            relation_pushforward(f, r, y, n) = exists(a: A, b: A) {
                                f(a) = y and f(b) = n and r(a, b)
                            }
                            let (a: A, b: A) satisfy {
                                f(a) = y and f(b) = n and r(a, b)
                            }
                            f(b) = f(m)
                            b = m
                            r(a, m)
                            forall(z: A) {
                                r(z, m) implies not p(z)
                            }
                            not p(a)
                            if q(y) {
                                f(a) = y
                                q(f(a))
                                p(a)
                                false
                            }
                            not q(y)
                        }
                    }
                    exists(minimal: B) {
                        q(minimal) and forall(y: B) {
                            relation_pushforward(f, r, y, minimal) implies not q(y)
                        }
                    }
                } else {
                    not exists(x: A) { p(x) }
                    forall(z: B) {
                        if relation_pushforward(f, r, z, y0) {
                            relation_pushforward(f, r, z, y0) = exists(a: A, b: A) {
                                f(a) = z and f(b) = y0 and r(a, b)
                            }
                            let (a: A, b: A) satisfy {
                                f(a) = z and f(b) = y0 and r(a, b)
                            }
                            q(y0)
                            f(b) = y0
                            q(f(b))
                            p(b)
                            exists(x: A) { p(x) }
                            false
                        }
                    }
                    q(y0) and forall(z: B) {
                        relation_pushforward(f, r, z, y0) implies not q(z)
                    }
                    exists(minimal: B) {
                        q(minimal) and forall(y: B) {
                            relation_pushforward(f, r, y, minimal) implies not q(y)
                        }
                    }
                }
            }
        }
        is_well_founded(relation_pushforward(f, r)) = forall(q: B -> Bool) {
            (exists(y: B) { q(y) }) implies exists(minimal: B) {
                q(minimal) and forall(y: B) {
                    relation_pushforward(f, r, y, minimal) implies not q(y)
                }
            }
        }
        is_well_founded(relation_pushforward(f, r))
    }
}

/// A bundled bijection transports a well-founded relation by pushforward.
theorem relation_pushforward_is_well_founded_of_bijection_map[A, B](e: Bijection[A, B], r: (A, A) -> Bool) {
    is_well_founded(r) implies is_well_founded(relation_pushforward(e.map, r))
} by {
    if is_well_founded(r) {
        bijection_map_is_bijection(e)
        relation_pushforward_is_well_founded_of_bijection(e.map, r)
        is_well_founded(relation_pushforward(e.map, r))
    }
}

/// A relation contained in a well-founded relation is well-founded.
theorem relation_subset_is_well_founded[T](r: (T, T) -> Bool, s: (T, T) -> Bool) {
    relation_subset(r, s) and is_well_founded(s) implies is_well_founded(r)
} by {
    if relation_subset(r, s) and is_well_founded(s) {
        is_well_founded(s) = forall(q: T -> Bool) {
            (exists(x: T) { q(x) }) implies exists(m: T) {
                q(m) and forall(y: T) {
                    s(y, m) implies not q(y)
                }
            }
        }
        forall(p: T -> Bool) {
            if exists(x: T) { p(x) } {
                exists(m: T) {
                    p(m) and forall(y: T) {
                        s(y, m) implies not p(y)
                    }
                }
                let m: T satisfy {
                    p(m) and forall(y: T) {
                        s(y, m) implies not p(y)
                    }
                }
                forall(y: T) {
                    if r(y, m) {
                        relation_subset_step(r, s, y, m)
                        s(y, m)
                        not p(y)
                    }
                }
                exists(n: T) {
                    p(n) and forall(y: T) {
                        r(y, n) implies not p(y)
                    }
                }
            }
        }
        is_well_founded(r)
    }
}

/// A relation controlled by the pullback of a well-founded relation is well-founded.
theorem relation_subset_pullback_is_well_founded[A, B](
    f: A -> B,
    r: (A, A) -> Bool,
    s: (B, B) -> Bool
) {
    relation_subset(r, relation_pullback(f, s)) and is_well_founded(s) implies is_well_founded(r)
} by {
    if relation_subset(r, relation_pullback(f, s)) and is_well_founded(s) {
        relation_pullback_is_well_founded(f, s)
        is_well_founded(relation_pullback(f, s))
        relation_subset_is_well_founded(r, relation_pullback(f, s))
        is_well_founded(r)
    }
}

/// A relation equivalent to the pullback of a well-founded relation is well-founded.
theorem relation_eq_pullback_is_well_founded[A, B](
    f: A -> B,
    r: (A, A) -> Bool,
    s: (B, B) -> Bool
) {
    relation_subset(r, relation_pullback(f, s)) and
    relation_subset(relation_pullback(f, s), r) and
    is_well_founded(s) implies is_well_founded(r)
} by {
    if relation_subset(r, relation_pullback(f, s)) and
        relation_subset(relation_pullback(f, s), r) and
        is_well_founded(s) {
        relation_subset_pullback_is_well_founded(f, r, s)
        is_well_founded(r)
    }
}

/// A relation whose pullback is contained in a well-founded relation is well-founded on the domain.
theorem relation_pullback_subset_is_well_founded[A, B](
    f: A -> B,
    r: (A, A) -> Bool,
    s: (B, B) -> Bool
) {
    relation_subset(relation_pullback(f, s), r) and is_well_founded(r) implies
    is_well_founded(relation_pullback(f, s))
} by {
    if relation_subset(relation_pullback(f, s), r) and is_well_founded(r) {
        relation_subset_is_well_founded(relation_pullback(f, s), r)
        is_well_founded(relation_pullback(f, s))
    }
}

/// A surjective pullback is well-founded only when the target relation is well-founded.
theorem relation_surjective_pullback_well_founded_reflects_target[A, B](f: A -> B, r: (B, B) -> Bool) {
    is_surjective_fn(f) and is_well_founded(relation_pullback(f, r)) implies is_well_founded(r)
} by {
    if is_surjective_fn(f) and is_well_founded(relation_pullback(f, r)) {
        is_surjective_fn(f) = forall(y: B) {
            exists(x: A) {
                f(x) = y
            }
        }
        is_well_founded(relation_pullback(f, r)) = forall(p: A -> Bool) {
            (exists(x: A) { p(x) }) implies exists(m: A) {
                p(m) and forall(y: A) {
                    relation_pullback(f, r, y, m) implies not p(y)
                }
            }
        }
        forall(q: B -> Bool) {
            if exists(x: B) { q(x) } {
                let p: A -> Bool = function(a: A) {
                    q(f(a))
                }
                let b0: B satisfy { q(b0) }
                exists(a0: A) {
                    f(a0) = b0
                }
                let a0: A satisfy {
                    f(a0) = b0
                }
                q(f(a0))
                p(a0)
                exists(a: A) {
                    p(a)
                }
                exists(m: A) {
                    p(m) and forall(y: A) {
                        relation_pullback(f, r, y, m) implies not p(y)
                    }
                }
                let m: A satisfy {
                    p(m) and forall(y: A) {
                        relation_pullback(f, r, y, m) implies not p(y)
                    }
                }
                let mb: B = f(m)
                q(mb)
                forall(yb: B) {
                    if r(yb, mb) {
                        exists(ya: A) {
                            f(ya) = yb
                        }
                        let ya: A satisfy {
                            f(ya) = yb
                        }
                        r(f(ya), f(m))
                        relation_pullback(f, r, ya, m)
                        not p(ya)
                        if q(yb) {
                            q(f(ya))
                            p(ya)
                            false
                        }
                        not q(yb)
                    }
                }
                q(mb) and forall(y: B) {
                    r(y, mb) implies not q(y)
                }
                exists(minimal: B) {
                    q(minimal) and forall(y: B) {
                        r(y, minimal) implies not q(y)
                    }
                }
            }
        }
        is_well_founded(r) = forall(q: B -> Bool) {
            (exists(x: B) { q(x) }) implies exists(m: B) {
                q(m) and forall(y: B) {
                    r(y, m) implies not q(y)
                }
            }
        }
        is_well_founded(r)
    }
}

/// Pulling back along a surjection preserves and reflects well-foundedness.
theorem relation_surjective_pullback_well_founded_iff[A, B](f: A -> B, r: (B, B) -> Bool) {
    is_surjective_fn(f) implies (is_well_founded(relation_pullback(f, r)) = is_well_founded(r))
} by {
    if is_surjective_fn(f) {
        if is_well_founded(relation_pullback(f, r)) {
            relation_surjective_pullback_well_founded_reflects_target(f, r)
            is_well_founded(r)
        }
        if is_well_founded(r) {
            relation_pullback_is_well_founded(f, r)
            is_well_founded(relation_pullback(f, r))
        }
        is_well_founded(relation_pullback(f, r)) = is_well_founded(r)
    }
}

/// Every relation is contained in the pullback of its pushforward along a map.
theorem relation_subset_pullback_pushforward[A, B](f: A -> B, r: (A, A) -> Bool) {
    relation_subset(r, relation_pullback(f, relation_pushforward(f, r)))
} by {
    forall(x: A, y: A) {
        if r(x, y) {
            relation_pushforward(f, r, f(x), f(y))
            relation_pullback(f, relation_pushforward(f, r), x, y)
        }
    }
}

/// A pushforward is well-founded only when the source relation is well-founded.
theorem relation_pushforward_well_founded_reflects_source[A, B](f: A -> B, r: (A, A) -> Bool) {
    is_well_founded(relation_pushforward(f, r)) implies is_well_founded(r)
} by {
    if is_well_founded(relation_pushforward(f, r)) {
        relation_subset_pullback_pushforward(f, r)
        relation_subset_pullback_is_well_founded(f, r, relation_pushforward(f, r))
        is_well_founded(r)
    }
}

/// An injective pushforward preserves and reflects well-foundedness.
theorem relation_injective_pushforward_well_founded_iff[A, B](f: A -> B, r: (A, A) -> Bool) {
    is_injective_fn(f) implies (is_well_founded(relation_pushforward(f, r)) = is_well_founded(r))
} by {
    if is_injective_fn(f) {
        if is_well_founded(relation_pushforward(f, r)) {
            relation_pushforward_well_founded_reflects_source(f, r)
            is_well_founded(r)
        }
        if is_well_founded(r) {
            relation_pushforward_is_well_founded_of_injection(f, r)
            is_well_founded(relation_pushforward(f, r))
        }
        is_well_founded(relation_pushforward(f, r)) = is_well_founded(r)
    }
}

/// Well-founded induction over predecessor-closed predicates.
theorem well_founded_induction[T](r: (T, T) -> Bool, p: T -> Bool) {
    is_well_founded(r) and
    forall(x: T) {
        (forall(y: T) { r(y, x) implies p(y) }) implies p(x)
    }
    implies
    forall(x: T) {
        p(x)
    }
} by {
    if is_well_founded(r) and
        forall(x: T) {
            (forall(y: T) { r(y, x) implies p(y) }) implies p(x)
        } {
        is_well_founded(r) = forall(q: T -> Bool) {
            (exists(x: T) { q(x) }) implies exists(m: T) {
                q(m) and forall(y: T) {
                    r(y, m) implies not q(y)
                }
            }
        }
        if not forall(x: T) { p(x) } {
            let bad: T -> Bool = function(x: T) {
                not p(x)
            }
            exists(x: T) {
                bad(x)
            }
            exists(m: T) {
                bad(m) and forall(y: T) {
                    r(y, m) implies not bad(y)
                }
            }
            let m: T satisfy {
                bad(m) and forall(y: T) {
                    r(y, m) implies not bad(y)
                }
            }
            forall(y: T) {
                if r(y, m) {
                    not bad(y)
                    p(y)
                }
            }
            p(m)
            bad(m)
            not p(m)
            false
        }
        forall(x: T) {
            p(x)
        }
    }
}

/// The intersection of a well-founded relation with any relation is well-founded.
theorem relation_intersection_left_is_well_founded[T](r: (T, T) -> Bool, s: (T, T) -> Bool) {
    is_well_founded(r) implies is_well_founded(relation_intersection(r, s))
} by {
    if is_well_founded(r) {
        relation_intersection_left_subset(r, s)
        relation_subset_is_well_founded(relation_intersection(r, s), r)
        is_well_founded(relation_intersection(r, s))
    }
}

/// The intersection of any relation with a well-founded relation is well-founded.
theorem relation_intersection_right_is_well_founded[T](r: (T, T) -> Bool, s: (T, T) -> Bool) {
    is_well_founded(s) implies is_well_founded(relation_intersection(r, s))
} by {
    if is_well_founded(s) {
        relation_intersection_right_subset(r, s)
        relation_subset_is_well_founded(relation_intersection(r, s), s)
        is_well_founded(relation_intersection(r, s))
    }
}

/// A relation equivalent to a well-founded relation is well-founded.
theorem relation_subset_both_is_well_founded[T](r: (T, T) -> Bool, s: (T, T) -> Bool) {
    relation_subset(r, s) and relation_subset(s, r) and is_well_founded(s) implies is_well_founded(r)
} by {
    if relation_subset(r, s) and relation_subset(s, r) and is_well_founded(s) {
        relation_subset_is_well_founded(r, s)
        is_well_founded(r)
    }
}

/// Well-founded induction gives the predicate at a chosen element.
theorem well_founded_induction_at[T](r: (T, T) -> Bool, p: T -> Bool, x: T) {
    is_well_founded(r) and
    forall(z: T) {
        (forall(y: T) { r(y, z) implies p(y) }) implies p(z)
    }
    implies
    p(x)
} by {
    if is_well_founded(r) and
        forall(z: T) {
            (forall(y: T) { r(y, z) implies p(y) }) implies p(z)
        } {
        well_founded_induction(r, p)
        forall(z: T) {
            p(z)
        }
        p(x)
    }
}

/// A Noetherian relation gives a maximal element of every nonempty predicate.
theorem noetherian_relation_has_maximal[T](r: (T, T) -> Bool, p: T -> Bool) {
    is_noetherian_relation(r) and exists(x: T) { p(x) } implies exists(m: T) {
        p(m) and forall(y: T) {
            r(m, y) implies not p(y)
        }
    }
} by {
    if is_noetherian_relation(r) and exists(x: T) { p(x) } {
        is_noetherian_relation(r)
        is_well_founded(relation_converse(r))
        is_well_founded(relation_converse(r)) = forall(q: T -> Bool) {
            (exists(x: T) { q(x) }) implies exists(m: T) {
                q(m) and forall(y: T) {
                    relation_converse(r, y, m) implies not q(y)
                }
            }
        }
        exists(m: T) {
            p(m) and forall(y: T) {
                relation_converse(r, y, m) implies not p(y)
            }
        }
        let m: T satisfy {
            p(m) and forall(y: T) {
                relation_converse(r, y, m) implies not p(y)
            }
        }
        forall(y: T) {
            if r(m, y) {
                relation_converse(r, y, m)
                not p(y)
            }
        }
        p(m) and forall(y: T) {
            r(m, y) implies not p(y)
        }
        exists(maximal: T) {
            p(maximal) and forall(y: T) {
                r(maximal, y) implies not p(y)
            }
        }
    }
}

/// A relation contained in a Noetherian relation is Noetherian.
theorem relation_subset_is_noetherian_relation[T](r: (T, T) -> Bool, s: (T, T) -> Bool) {
    relation_subset(r, s) and is_noetherian_relation(s) implies is_noetherian_relation(r)
} by {
    if relation_subset(r, s) and is_noetherian_relation(s) {
        relation_converse_monotone(r, s)
        relation_subset(relation_converse(r), relation_converse(s))
        is_well_founded(relation_converse(s))
        relation_subset_is_well_founded(relation_converse(r), relation_converse(s))
        is_well_founded(relation_converse(r))
        is_noetherian_relation(r)
    }
}

/// The intersection of a Noetherian relation with any relation is Noetherian.
theorem relation_intersection_left_is_noetherian_relation[T](r: (T, T) -> Bool, s: (T, T) -> Bool) {
    is_noetherian_relation(r) implies is_noetherian_relation(relation_intersection(r, s))
} by {
    if is_noetherian_relation(r) {
        relation_intersection_left_subset(r, s)
        relation_subset_is_noetherian_relation(relation_intersection(r, s), r)
        is_noetherian_relation(relation_intersection(r, s))
    }
}

/// The intersection of any relation with a Noetherian relation is Noetherian.
theorem relation_intersection_right_is_noetherian_relation[T](r: (T, T) -> Bool, s: (T, T) -> Bool) {
    is_noetherian_relation(s) implies is_noetherian_relation(relation_intersection(r, s))
} by {
    if is_noetherian_relation(s) {
        relation_intersection_right_subset(r, s)
        relation_subset_is_noetherian_relation(relation_intersection(r, s), s)
        is_noetherian_relation(relation_intersection(r, s))
    }
}

/// Noetherianity is invariant under mutual relation inclusion.
theorem relation_subset_both_is_noetherian_relation[T](r: (T, T) -> Bool, s: (T, T) -> Bool) {
    relation_subset(r, s) and relation_subset(s, r) and is_noetherian_relation(s) implies
    is_noetherian_relation(r)
} by {
    if relation_subset(r, s) and relation_subset(s, r) and is_noetherian_relation(s) {
        relation_subset_is_noetherian_relation(r, s)
        is_noetherian_relation(r)
    }
}

/// The pullback of a Noetherian relation along any function is Noetherian.
theorem relation_pullback_is_noetherian_relation[A, B](f: A -> B, r: (B, B) -> Bool) {
    is_noetherian_relation(r) implies is_noetherian_relation(relation_pullback(f, r))
} by {
    if is_noetherian_relation(r) {
        is_well_founded(relation_converse(r))
        relation_pullback_is_well_founded(f, relation_converse(r))
        is_well_founded(relation_pullback(f, relation_converse(r)))
        relation_pullback_converse(f, r)
        is_well_founded(relation_converse(relation_pullback(f, r)))
        is_noetherian_relation(relation_pullback(f, r))
    }
}

/// A relation controlled by the pullback of a Noetherian relation is Noetherian.
theorem relation_subset_pullback_is_noetherian_relation[A, B](
    f: A -> B,
    r: (A, A) -> Bool,
    s: (B, B) -> Bool
) {
    relation_subset(r, relation_pullback(f, s)) and is_noetherian_relation(s) implies
    is_noetherian_relation(r)
} by {
    if relation_subset(r, relation_pullback(f, s)) and is_noetherian_relation(s) {
        relation_pullback_is_noetherian_relation(f, s)
        is_noetherian_relation(relation_pullback(f, s))
        relation_subset_is_noetherian_relation(r, relation_pullback(f, s))
        is_noetherian_relation(r)
    }
}

/// A relation equivalent to the pullback of a Noetherian relation is Noetherian.
theorem relation_eq_pullback_is_noetherian_relation[A, B](
    f: A -> B,
    r: (A, A) -> Bool,
    s: (B, B) -> Bool
) {
    relation_subset(r, relation_pullback(f, s)) and
    relation_subset(relation_pullback(f, s), r) and
    is_noetherian_relation(s) implies is_noetherian_relation(r)
} by {
    if relation_subset(r, relation_pullback(f, s)) and
        relation_subset(relation_pullback(f, s), r) and
        is_noetherian_relation(s) {
        relation_subset_pullback_is_noetherian_relation(f, r, s)
        is_noetherian_relation(r)
    }
}

/// A relation whose pullback is contained in a Noetherian relation is Noetherian on the domain.
theorem relation_pullback_subset_is_noetherian_relation[A, B](
    f: A -> B,
    r: (A, A) -> Bool,
    s: (B, B) -> Bool
) {
    relation_subset(relation_pullback(f, s), r) and is_noetherian_relation(r) implies
    is_noetherian_relation(relation_pullback(f, s))
} by {
    if relation_subset(relation_pullback(f, s), r) and is_noetherian_relation(r) {
        relation_subset_is_noetherian_relation(relation_pullback(f, s), r)
        is_noetherian_relation(relation_pullback(f, s))
    }
}

/// A surjective pullback is Noetherian only when the target relation is Noetherian.
theorem relation_surjective_pullback_noetherian_reflects_target[A, B](f: A -> B, r: (B, B) -> Bool) {
    is_surjective_fn(f) and is_noetherian_relation(relation_pullback(f, r)) implies is_noetherian_relation(r)
} by {
    if is_surjective_fn(f) and is_noetherian_relation(relation_pullback(f, r)) {
        is_well_founded(relation_converse(relation_pullback(f, r)))
        relation_pullback_converse(f, r)
        is_well_founded(relation_pullback(f, relation_converse(r)))
        relation_surjective_pullback_well_founded_reflects_target(f, relation_converse(r))
        is_well_founded(relation_converse(r))
        is_noetherian_relation(r)
    }
}

/// Pulling back along a surjection preserves and reflects Noetherianity.
theorem relation_surjective_pullback_noetherian_iff[A, B](f: A -> B, r: (B, B) -> Bool) {
    is_surjective_fn(f) implies
    (is_noetherian_relation(relation_pullback(f, r)) = is_noetherian_relation(r))
} by {
    if is_surjective_fn(f) {
        if is_noetherian_relation(relation_pullback(f, r)) {
            relation_surjective_pullback_noetherian_reflects_target(f, r)
            is_noetherian_relation(r)
        }
        if is_noetherian_relation(r) {
            relation_pullback_is_noetherian_relation(f, r)
            is_noetherian_relation(relation_pullback(f, r))
        }
        is_noetherian_relation(relation_pullback(f, r)) = is_noetherian_relation(r)
    }
}

/// The pushforward of a Noetherian relation along a bijection is Noetherian.
theorem relation_pushforward_is_noetherian_relation_of_bijection[A, B](f: A -> B, r: (A, A) -> Bool) {
    is_bijection_fn(f) and is_noetherian_relation(r) implies is_noetherian_relation(relation_pushforward(f, r))
} by {
    if is_bijection_fn(f) and is_noetherian_relation(r) {
        is_well_founded(relation_converse(r))
        relation_pushforward_is_well_founded_of_bijection(f, relation_converse(r))
        is_well_founded(relation_pushforward(f, relation_converse(r)))
        relation_pushforward_converse(f, r)
        is_well_founded(relation_converse(relation_pushforward(f, r)))
        is_noetherian_relation(relation_pushforward(f, r))
    }
}

/// The pushforward of a Noetherian relation along an injection is Noetherian.
theorem relation_pushforward_is_noetherian_relation_of_injection[A, B](f: A -> B, r: (A, A) -> Bool) {
    is_injective_fn(f) and is_noetherian_relation(r) implies is_noetherian_relation(relation_pushforward(f, r))
} by {
    if is_injective_fn(f) and is_noetherian_relation(r) {
        is_well_founded(relation_converse(r))
        relation_pushforward_is_well_founded_of_injection(f, relation_converse(r))
        is_well_founded(relation_pushforward(f, relation_converse(r)))
        relation_pushforward_converse(f, r)
        is_well_founded(relation_converse(relation_pushforward(f, r)))
        is_noetherian_relation(relation_pushforward(f, r))
    }
}

/// A bundled bijection transports a Noetherian relation by pushforward.
theorem relation_pushforward_is_noetherian_relation_of_bijection_map[A, B](e: Bijection[A, B], r: (A, A) -> Bool) {
    is_noetherian_relation(r) implies is_noetherian_relation(relation_pushforward(e.map, r))
} by {
    if is_noetherian_relation(r) {
        bijection_map_is_bijection(e)
        relation_pushforward_is_noetherian_relation_of_bijection(e.map, r)
        is_noetherian_relation(relation_pushforward(e.map, r))
    }
}

/// A pushforward is Noetherian only when the source relation is Noetherian.
theorem relation_pushforward_noetherian_reflects_source[A, B](f: A -> B, r: (A, A) -> Bool) {
    is_noetherian_relation(relation_pushforward(f, r)) implies is_noetherian_relation(r)
} by {
    if is_noetherian_relation(relation_pushforward(f, r)) {
        is_well_founded(relation_converse(relation_pushforward(f, r)))
        relation_pushforward_converse(f, r)
        is_well_founded(relation_pushforward(f, relation_converse(r)))
        relation_pushforward_well_founded_reflects_source(f, relation_converse(r))
        is_well_founded(relation_converse(r))
        is_noetherian_relation(r)
    }
}

/// An injective pushforward preserves and reflects Noetherianity.
theorem relation_injective_pushforward_noetherian_iff[A, B](f: A -> B, r: (A, A) -> Bool) {
    is_injective_fn(f) implies
    (is_noetherian_relation(relation_pushforward(f, r)) = is_noetherian_relation(r))
} by {
    if is_injective_fn(f) {
        if is_noetherian_relation(relation_pushforward(f, r)) {
            relation_pushforward_noetherian_reflects_source(f, r)
            is_noetherian_relation(r)
        }
        if is_noetherian_relation(r) {
            relation_pushforward_is_noetherian_relation_of_injection(f, r)
            is_noetherian_relation(relation_pushforward(f, r))
        }
        is_noetherian_relation(relation_pushforward(f, r)) = is_noetherian_relation(r)
    }
}

/// Noetherian induction over successor-closed predicates.
theorem noetherian_relation_induction[T](r: (T, T) -> Bool, p: T -> Bool) {
    is_noetherian_relation(r) and is_successor_closed_predicate(r, p)
    implies
    forall(x: T) {
        p(x)
    }
} by {
    if is_noetherian_relation(r) {
        if is_successor_closed_predicate(r, p) {
            is_noetherian_relation(r)
            is_well_founded(relation_converse(r))
            is_successor_closed_predicate(r, p) = forall(a: T) {
                (forall(b: T) { r(a, b) implies p(b) }) implies p(a)
            }
            forall(x: T) {
                if forall(y: T) { relation_converse(r, y, x) implies p(y) } {
                    forall(y: T) {
                        if r(x, y) {
                            relation_converse(r, y, x)
                            p(y)
                        }
                    }
                    forall(y: T) { r(x, y) implies p(y) }
                    successor_closed_predicate_at(r, p, x)
                    p(x)
                }
            }
            well_founded_induction(relation_converse(r), p)
            forall(x: T) {
                well_founded_induction_at(relation_converse(r), p, x)
                p(x)
            }
        }
    }
}

/// Noetherian induction gives the predicate at a chosen element.
theorem noetherian_relation_induction_at[T](r: (T, T) -> Bool, p: T -> Bool, x: T) {
    is_noetherian_relation(r) and is_successor_closed_predicate(r, p)
    implies
    p(x)
} by {
    if is_noetherian_relation(r) and is_successor_closed_predicate(r, p) {
        noetherian_relation_induction(r, p)
        forall(z: T) {
            p(z)
        }
        p(x)
    }
}


/// Recursive solutions over a well-founded relation agree pointwise.
theorem well_founded_recursive_solution_eq_at[T, U](
    r: (T, T) -> Bool,
    step: (T, T -> U) -> U,
    f: T -> U,
    g: T -> U,
    x: T
) {
    is_well_founded(r) and
    is_predecessor_extensional_recursive_step(r, step) and
    is_well_founded_recursive_solution(r, step, f) and
    is_well_founded_recursive_solution(r, step, g) implies
    f(x) = g(x)
} by {
    if is_well_founded(r) and
        is_predecessor_extensional_recursive_step(r, step) and
        is_well_founded_recursive_solution(r, step, f) and
        is_well_founded_recursive_solution(r, step, g) {
        let p: T -> Bool = function(z: T) {
            f(z) = g(z)
        }
        forall(z: T) {
            if forall(y: T) { r(y, z) implies p(y) } {
                functions_agree_on_predecessors(r, z, f, g)
                is_predecessor_extensional_recursive_step(r, step) =
                    forall(a: T, h: T -> U, k: T -> U) {
                        functions_agree_on_predecessors(r, a, h, k) implies step(a, h) = step(a, k)
                    }
                step(z, f) = step(z, g)
                is_well_founded_recursive_solution(r, step, f) = forall(a: T) {
                    f(a) = step(a, f)
                }
                is_well_founded_recursive_solution(r, step, g) = forall(a: T) {
                    g(a) = step(a, g)
                }
                f(z) = step(z, f)
                g(z) = step(z, g)
                f(z) = g(z)
                p(z)
            }
        }
        well_founded_induction_at(r, p, x)
        p(x)
        f(x) = g(x)
    }
}

/// If a well-founded recursive equation has a solution, then it has exactly one solution.
theorem well_founded_recursive_solution_exists_unique[T, U](
    r: (T, T) -> Bool,
    step: (T, T -> U) -> U
) {
    is_well_founded(r) and
    is_predecessor_extensional_recursive_step(r, step) and
    exists(f: T -> U) {
        is_well_founded_recursive_solution(r, step, f)
    } implies
    exists_unique(well_founded_recursive_solution_predicate(r, step))
} by {
    if is_well_founded(r) and
        is_predecessor_extensional_recursive_step(r, step) and
        exists(f: T -> U) {
            is_well_founded_recursive_solution(r, step, f)
        } {
        forall(f: T -> U, g: T -> U) {
            if well_founded_recursive_solution_predicate(r, step, f) and
                well_founded_recursive_solution_predicate(r, step, g) {
                is_well_founded_recursive_solution(r, step, f)
                is_well_founded_recursive_solution(r, step, g)
                forall(x: T) {
                    well_founded_recursive_solution_eq_at(r, step, f, g, x)
                    f(x) = g(x)
                }
                function_extensionality(f, g)
                f = g
            }
        }
        exists(f: T -> U) {
            well_founded_recursive_solution_predicate(r, step, f)
        }
        exists_unique_of_exists_and_pairwise(well_founded_recursive_solution_predicate(r, step))
        exists_unique(well_founded_recursive_solution_predicate(r, step))
    }
}

/// Recursive solutions over a well-founded relation are equal as functions.
theorem well_founded_recursive_solution_eq[T, U](
    r: (T, T) -> Bool,
    step: (T, T -> U) -> U,
    f: T -> U,
    g: T -> U
) {
    is_well_founded(r) and
    is_predecessor_extensional_recursive_step(r, step) and
    is_well_founded_recursive_solution(r, step, f) and
    is_well_founded_recursive_solution(r, step, g) implies
    f = g
} by {
    if is_well_founded(r) and
        is_predecessor_extensional_recursive_step(r, step) and
        is_well_founded_recursive_solution(r, step, f) and
        is_well_founded_recursive_solution(r, step, g) {
        forall(x: T) {
            well_founded_recursive_solution_eq_at(r, step, f, g, x)
            f(x) = g(x)
        }
        function_extensionality(f, g)
        f = g
    }
}

/// The chosen recursive solution satisfies the recursive equation whenever a solution exists.
theorem choose_well_founded_recursive_solution_is_solution[T, U](
    r: (T, T) -> Bool,
    step: (T, T -> U) -> U,
    default: U
) {
    is_well_founded(r) and
    is_predecessor_extensional_recursive_step(r, step) and
    exists(f: T -> U) {
        is_well_founded_recursive_solution(r, step, f)
    } implies
    is_well_founded_recursive_solution(r, step, choose_well_founded_recursive_solution(r, step, default))
} by {
    if is_well_founded(r) and
        is_predecessor_extensional_recursive_step(r, step) and
        exists(f: T -> U) {
            is_well_founded_recursive_solution(r, step, f)
        } {
        well_founded_recursive_solution_exists_unique(r, step)
        exists_unique(well_founded_recursive_solution_predicate(r, step))
        exists(f: T -> U) {
            well_founded_recursive_solution_predicate(r, step, f)
        }
        choose_or_default_spec(
            well_founded_recursive_solution_predicate(r, step),
            well_founded_recursive_default_function[T, U](default)
        )
        well_founded_recursive_solution_predicate(
            r,
            step,
            choose_well_founded_recursive_solution(r, step, default)
        )
        is_well_founded_recursive_solution(r, step, choose_well_founded_recursive_solution(r, step, default))
    }
}

/// The chosen recursive solution equals any supplied recursive solution.
theorem choose_well_founded_recursive_solution_eq_solution[T, U](
    r: (T, T) -> Bool,
    step: (T, T -> U) -> U,
    default: U,
    f: T -> U
) {
    is_well_founded(r) and
    is_predecessor_extensional_recursive_step(r, step) and
    is_well_founded_recursive_solution(r, step, f) implies
    choose_well_founded_recursive_solution(r, step, default) = f
} by {
    if is_well_founded(r) and
        is_predecessor_extensional_recursive_step(r, step) and
        is_well_founded_recursive_solution(r, step, f) {
        exists(g: T -> U) {
            is_well_founded_recursive_solution(r, step, g)
        }
        well_founded_recursive_solution_exists_unique(r, step)
        exists_unique(well_founded_recursive_solution_predicate(r, step))
        well_founded_recursive_solution_predicate(r, step, f)
        choose_or_default_unique_eq(
            well_founded_recursive_solution_predicate(r, step),
            well_founded_recursive_default_function[T, U](default),
            f
        )
        choose_well_founded_recursive_solution(r, step, default) = f
    }
}

/// The chosen recursive solution does not depend on the default value when a solution exists.
theorem choose_well_founded_recursive_solution_default_independent[T, U](
    r: (T, T) -> Bool,
    step: (T, T -> U) -> U,
    default1: U,
    default2: U
) {
    is_well_founded(r) and
    is_predecessor_extensional_recursive_step(r, step) and
    exists(f: T -> U) {
        is_well_founded_recursive_solution(r, step, f)
    } implies
    choose_well_founded_recursive_solution(r, step, default1) =
    choose_well_founded_recursive_solution(r, step, default2)
} by {
    if is_well_founded(r) and
        is_predecessor_extensional_recursive_step(r, step) and
        exists(f: T -> U) {
            is_well_founded_recursive_solution(r, step, f)
        } {
        well_founded_recursive_solution_exists_unique(r, step)
        exists_unique(well_founded_recursive_solution_predicate(r, step))
        choose_or_default_unique_default_eq(
            well_founded_recursive_solution_predicate(r, step),
            well_founded_recursive_default_function[T, U](default1),
            well_founded_recursive_default_function[T, U](default2)
        )
        choose_well_founded_recursive_solution(r, step, default1) =
        choose_well_founded_recursive_solution(r, step, default2)
    }
}

/// The chosen recursive solution satisfies the recursive equation at a chosen element.
theorem choose_well_founded_recursive_solution_eq_step_at[T, U](
    r: (T, T) -> Bool,
    step: (T, T -> U) -> U,
    default: U,
    x: T
) {
    is_well_founded(r) and
    is_predecessor_extensional_recursive_step(r, step) and
    exists(f: T -> U) {
        is_well_founded_recursive_solution(r, step, f)
    } implies
    choose_well_founded_recursive_solution(r, step, default)(x) =
    step(x, choose_well_founded_recursive_solution(r, step, default))
} by {
    if is_well_founded(r) and
        is_predecessor_extensional_recursive_step(r, step) and
        exists(f: T -> U) {
            is_well_founded_recursive_solution(r, step, f)
        } {
        choose_well_founded_recursive_solution_is_solution(r, step, default)
        is_well_founded_recursive_solution(r, step, choose_well_founded_recursive_solution(r, step, default)) =
            forall(y: T) {
                choose_well_founded_recursive_solution(r, step, default)(y) =
                step(y, choose_well_founded_recursive_solution(r, step, default))
            }
        choose_well_founded_recursive_solution(r, step, default)(x) =
        step(x, choose_well_founded_recursive_solution(r, step, default))
    }
}

/// The nonrecursive step depends only on predecessor values in the trivial way.
theorem well_founded_nonrecursive_step_is_predecessor_extensional[T, U](
    r: (T, T) -> Bool,
    value: T -> U
) {
    is_predecessor_extensional_recursive_step(r, well_founded_nonrecursive_step(value))
} by {
    forall(x: T, f: T -> U, g: T -> U) {
        if functions_agree_on_predecessors(r, x, f, g) {
            well_founded_nonrecursive_step(value, x, f) = value(x)
            well_founded_nonrecursive_step(value, x, g) = value(x)
            well_founded_nonrecursive_step(value, x, f) =
                well_founded_nonrecursive_step(value, x, g)
        }
    }
}

/// The direct function is a solution of the nonrecursive equation.
theorem well_founded_nonrecursive_function_is_solution[T, U](
    r: (T, T) -> Bool,
    value: T -> U
) {
    is_well_founded_recursive_solution(
        r,
        well_founded_nonrecursive_step(value),
        well_founded_nonrecursive_function(value)
    )
} by {
    is_well_founded_recursive_solution(
        r,
        well_founded_nonrecursive_step(value),
        well_founded_nonrecursive_function(value)
    ) =
        forall(x: T) {
            well_founded_nonrecursive_function(value, x) =
            well_founded_nonrecursive_step(value, x, well_founded_nonrecursive_function(value))
        }
    forall(x: T) {
        well_founded_nonrecursive_function(value, x) = value(x)
        well_founded_nonrecursive_step(value, x, well_founded_nonrecursive_function(value)) = value(x)
        well_founded_nonrecursive_function(value, x) =
            well_founded_nonrecursive_step(value, x, well_founded_nonrecursive_function(value))
    }
}

/// The nonrecursive equation has a solution.
theorem well_founded_nonrecursive_solution_exists[T, U](
    r: (T, T) -> Bool,
    value: T -> U
) {
    exists(f: T -> U) {
        is_well_founded_recursive_solution(r, well_founded_nonrecursive_step(value), f)
    }
} by {
    well_founded_nonrecursive_function_is_solution(r, value)
    exists(f: T -> U) {
        f = well_founded_nonrecursive_function(value) and
        is_well_founded_recursive_solution(r, well_founded_nonrecursive_step(value), f)
    }
}

/// The nonrecursive equation has a unique solution over a well-founded relation.
theorem well_founded_nonrecursive_solution_exists_unique[T, U](
    r: (T, T) -> Bool,
    value: T -> U
) {
    is_well_founded(r) implies
    exists_unique(well_founded_recursive_solution_predicate(r, well_founded_nonrecursive_step(value)))
} by {
    if is_well_founded(r) {
        well_founded_nonrecursive_step_is_predecessor_extensional(r, value)
        well_founded_nonrecursive_solution_exists(r, value)
        well_founded_recursive_solution_exists_unique(r, well_founded_nonrecursive_step(value))
        exists_unique(well_founded_recursive_solution_predicate(r, well_founded_nonrecursive_step(value)))
    }
}

/// Every solution of the nonrecursive equation is the direct function.
theorem well_founded_nonrecursive_solution_eq_function[T, U](
    r: (T, T) -> Bool,
    value: T -> U,
    f: T -> U
) {
    is_well_founded(r) and
    is_well_founded_recursive_solution(r, well_founded_nonrecursive_step(value), f) implies
    f = well_founded_nonrecursive_function(value)
} by {
    if is_well_founded(r) and
        is_well_founded_recursive_solution(r, well_founded_nonrecursive_step(value), f) {
        well_founded_nonrecursive_step_is_predecessor_extensional(r, value)
        well_founded_nonrecursive_function_is_solution(r, value)
        well_founded_recursive_solution_eq(
            r,
            well_founded_nonrecursive_step(value),
            f,
            well_founded_nonrecursive_function(value)
        )
        f = well_founded_nonrecursive_function(value)
    }
}

/// The chosen solution of the nonrecursive equation is the direct function.
theorem choose_well_founded_nonrecursive_solution_eq_function[T, U](
    r: (T, T) -> Bool,
    value: T -> U,
    default: U
) {
    is_well_founded(r) implies
    choose_well_founded_recursive_solution(r, well_founded_nonrecursive_step(value), default) =
    well_founded_nonrecursive_function(value)
} by {
    if is_well_founded(r) {
        well_founded_nonrecursive_step_is_predecessor_extensional(r, value)
        well_founded_nonrecursive_function_is_solution(r, value)
        choose_well_founded_recursive_solution_eq_solution(
            r,
            well_founded_nonrecursive_step(value),
            default,
            well_founded_nonrecursive_function(value)
        )
        choose_well_founded_recursive_solution(r, well_founded_nonrecursive_step(value), default) =
        well_founded_nonrecursive_function(value)
    }
}

/// The chosen solution of the nonrecursive equation satisfies the equation.
theorem choose_well_founded_nonrecursive_solution_is_solution[T, U](
    r: (T, T) -> Bool,
    value: T -> U,
    default: U
) {
    is_well_founded(r) implies
    is_well_founded_recursive_solution(
        r,
        well_founded_nonrecursive_step(value),
        choose_well_founded_recursive_solution(r, well_founded_nonrecursive_step(value), default)
    )
} by {
    if is_well_founded(r) {
        choose_well_founded_nonrecursive_solution_eq_function(r, value, default)
        well_founded_nonrecursive_function_is_solution(r, value)
        is_well_founded_recursive_solution(
            r,
            well_founded_nonrecursive_step(value),
            choose_well_founded_recursive_solution(r, well_founded_nonrecursive_step(value), default)
        )
    }
}

/// The chosen solution of the nonrecursive equation has the assigned value.
theorem choose_well_founded_nonrecursive_solution_value[T, U](
    r: (T, T) -> Bool,
    value: T -> U,
    default: U,
    x: T
) {
    is_well_founded(r) implies
    choose_well_founded_recursive_solution(r, well_founded_nonrecursive_step(value), default)(x) =
    value(x)
} by {
    if is_well_founded(r) {
        choose_well_founded_nonrecursive_solution_eq_function(r, value, default)
        well_founded_nonrecursive_function(value, x) = value(x)
        choose_well_founded_recursive_solution(r, well_founded_nonrecursive_step(value), default)(x) =
        value(x)
    }
}

/// The chosen solution of the nonrecursive equation is independent of the default value.
theorem choose_well_founded_nonrecursive_solution_default_independent[T, U](
    r: (T, T) -> Bool,
    value: T -> U,
    default1: U,
    default2: U
) {
    is_well_founded(r) implies
    choose_well_founded_recursive_solution(r, well_founded_nonrecursive_step(value), default1) =
    choose_well_founded_recursive_solution(r, well_founded_nonrecursive_step(value), default2)
} by {
    if is_well_founded(r) {
        choose_well_founded_nonrecursive_solution_eq_function(r, value, default1)
        choose_well_founded_nonrecursive_solution_eq_function(r, value, default2)
        choose_well_founded_recursive_solution(r, well_founded_nonrecursive_step(value), default1) =
        choose_well_founded_recursive_solution(r, well_founded_nonrecursive_step(value), default2)
    }
}

/// A nonempty predicate on natural numbers has a least element for the strict order.
theorem nat_lt_relation_has_minimal(p: Nat -> Bool) {
    (exists(x: Nat) { p(x) }) implies exists(m: Nat) {
        p(m) and forall(y: Nat) {
            nat_lt_relation(y, m) implies not p(y)
        }
    }
} by {
    if exists(x: Nat) { p(x) } {
        if not exists(m: Nat) {
            p(m) and forall(y: Nat) {
                nat_lt_relation(y, m) implies not p(y)
            }
        } {
            let q: Nat -> Bool = function(n: Nat) {
                not p(n)
            }
            forall(k: Nat) {
                if true_below(q, k) {
                    if p(k) {
                        forall(y: Nat) {
                            if nat_lt_relation(y, k) {
                                y < k
                                true_below(q, k) = forall(z: Nat) {
                                    z < k implies q(z)
                                }
                                q(y)
                                not p(y)
                            }
                        }
                        exists(m: Nat) {
                            p(m) and forall(y: Nat) {
                                nat_lt_relation(y, m) implies not p(y)
                            }
                        }
                        false
                    }
                    not p(k)
                    q(k)
                }
            }
            strong_induction(q)
            forall(n: Nat) {
                q(n)
            }
            let x: Nat satisfy {
                p(x)
            }
            q(x)
            not p(x)
            false
        }
        exists(m: Nat) {
            p(m) and forall(y: Nat) {
                nat_lt_relation(y, m) implies not p(y)
            }
        }
    }
}

/// A nonempty predicate on natural numbers has a least witness.
theorem exists_least_nat_witness(p: Nat -> Bool) {
    (exists(x: Nat) { p(x) }) implies exists(m: Nat) {
        is_least_nat_witness(p, m)
    }
} by {
    if exists(x: Nat) { p(x) } {
        nat_lt_relation_has_minimal(p)
        let m: Nat satisfy {
            p(m) and forall(y: Nat) {
                nat_lt_relation(y, m) implies not p(y)
            }
        }
        forall(y: Nat) {
            if y < m {
                nat_lt_relation(y, m)
                not p(y)
            }
        }
        is_least_nat_witness(p, m)
        exists(k: Nat) {
            is_least_nat_witness(p, k)
        }
    }
}

/// A least witness is no larger than any other witness.
theorem least_nat_witness_le_of_witness(p: Nat -> Bool, m: Nat, n: Nat) {
    is_least_nat_witness(p, m) and p(n) implies m <= n
} by {
    if is_least_nat_witness(p, m) and p(n) {
        is_least_nat_witness(p, m) = (
            p(m) and forall(y: Nat) {
                y < m implies not p(y)
            }
        )
        forall(y: Nat) {
            y < m implies not p(y)
        }
        if n < m {
            not p(n)
            false
        }
        lt_or_lte(n, m)
        m <= n
    }
}

/// If a predicate fails somewhere, it has a least counterexample.
theorem exists_least_nat_counterexample(p: Nat -> Bool) {
    (exists(x: Nat) { not p(x) }) implies exists(m: Nat) {
        is_least_nat_counterexample(p, m)
    }
} by {
    if exists(x: Nat) { not p(x) } {
        let q: Nat -> Bool = function(n: Nat) {
            not p(n)
        }
        exists(x: Nat) {
            q(x)
        }
        exists_least_nat_witness(q)
        let m: Nat satisfy {
            is_least_nat_witness(q, m)
        }
        is_least_nat_witness(q, m) = (
            q(m) and forall(y: Nat) {
                y < m implies not q(y)
            }
        )
        not p(m)
        forall(y: Nat) {
            if y < m {
                not q(y)
                p(y)
            }
        }
        is_least_nat_counterexample(p, m)
        exists(k: Nat) {
            is_least_nat_counterexample(p, k)
        }
    }
}

/// A least counterexample is no larger than any other counterexample.
theorem least_nat_counterexample_le_of_counterexample(p: Nat -> Bool, m: Nat, n: Nat) {
    is_least_nat_counterexample(p, m) and not p(n) implies m <= n
} by {
    if is_least_nat_counterexample(p, m) and not p(n) {
        is_least_nat_counterexample(p, m) = (
            not p(m) and forall(y: Nat) {
                y < m implies p(y)
            }
        )
        forall(y: Nat) {
            y < m implies p(y)
        }
        if n < m {
            p(n)
            false
        }
        lt_or_lte(n, m)
        m <= n
    }
}

/// Induction over the strict natural order.
theorem nat_lt_relation_induction(p: Nat -> Bool) {
    forall(n: Nat) {
        (forall(k: Nat) { nat_lt_relation(k, n) implies p(k) }) implies p(n)
    }
    implies
    forall(n: Nat) {
        p(n)
    }
} by {
    if forall(n: Nat) {
        (forall(k: Nat) { nat_lt_relation(k, n) implies p(k) }) implies p(n)
    } {
        forall(n: Nat) {
            if true_below(p, n) {
                forall(k: Nat) {
                    if nat_lt_relation(k, n) {
                        k < n
                        true_below(p, n) = forall(x: Nat) {
                            x < n implies p(x)
                        }
                        p(k)
                    }
                }
                p(n)
            }
        }
        strong_induction(p)
        forall(n: Nat) {
            p(n)
        }
    }
}

/// Induction over the strict natural order gives the predicate at a chosen number.
theorem nat_lt_relation_induction_at(p: Nat -> Bool, n: Nat) {
    forall(x: Nat) {
        (forall(k: Nat) { nat_lt_relation(k, x) implies p(k) }) implies p(x)
    }
    implies
    p(n)
} by {
    if forall(x: Nat) {
        (forall(k: Nat) { nat_lt_relation(k, x) implies p(k) }) implies p(x)
    } {
        nat_lt_relation_induction(p)
        forall(x: Nat) {
            p(x)
        }
        p(n)
    }
}

/// Recursive solutions over the strict natural order agree at a chosen number.
theorem nat_recursive_solution_eq_at[U](
    step: (Nat, Nat -> U) -> U,
    f: Nat -> U,
    g: Nat -> U,
    n: Nat
) {
    is_predecessor_extensional_recursive_step(nat_lt_relation, step) and
    is_nat_recursive_solution(step, f) and
    is_nat_recursive_solution(step, g) implies
    f(n) = g(n)
} by {
    if is_predecessor_extensional_recursive_step(nat_lt_relation, step) and
        is_nat_recursive_solution(step, f) and
        is_nat_recursive_solution(step, g) {
        let p: Nat -> Bool = function(z: Nat) {
            f(z) = g(z)
        }
        forall(z: Nat) {
            if forall(y: Nat) { nat_lt_relation(y, z) implies p(y) } {
                functions_agree_on_predecessors(nat_lt_relation, z, f, g)
                is_predecessor_extensional_recursive_step(nat_lt_relation, step) =
                    forall(a: Nat, h: Nat -> U, k: Nat -> U) {
                        functions_agree_on_predecessors(nat_lt_relation, a, h, k) implies
                        step(a, h) = step(a, k)
                    }
                step(z, f) = step(z, g)
                is_nat_recursive_solution(step, f)
                is_nat_recursive_solution(step, g)
                is_well_founded_recursive_solution(nat_lt_relation, step, f) = forall(a: Nat) {
                    f(a) = step(a, f)
                }
                is_well_founded_recursive_solution(nat_lt_relation, step, g) = forall(a: Nat) {
                    g(a) = step(a, g)
                }
                f(z) = step(z, f)
                g(z) = step(z, g)
                f(z) = g(z)
                p(z)
            }
        }
        nat_lt_relation_induction_at(p, n)
        p(n)
        f(n) = g(n)
    }
}

/// Recursive solutions over the strict natural order are equal as functions.
theorem nat_recursive_solution_eq[U](step: (Nat, Nat -> U) -> U, f: Nat -> U, g: Nat -> U) {
    is_predecessor_extensional_recursive_step(nat_lt_relation, step) and
    is_nat_recursive_solution(step, f) and
    is_nat_recursive_solution(step, g) implies
    f = g
} by {
    if is_predecessor_extensional_recursive_step(nat_lt_relation, step) and
        is_nat_recursive_solution(step, f) and
        is_nat_recursive_solution(step, g) {
        forall(n: Nat) {
            nat_recursive_solution_eq_at(step, f, g, n)
            f(n) = g(n)
        }
        function_extensionality(f, g)
        f = g
    }
}

/// If a strict-natural-order recursive equation has a solution, then it has exactly one solution.
theorem nat_recursive_solution_exists_unique[U](step: (Nat, Nat -> U) -> U) {
    is_predecessor_extensional_recursive_step(nat_lt_relation, step) and
    exists(f: Nat -> U) {
        is_nat_recursive_solution(step, f)
    } implies
    exists_unique(nat_recursive_solution_predicate(step))
} by {
    if is_predecessor_extensional_recursive_step(nat_lt_relation, step) and
        exists(f: Nat -> U) {
            is_nat_recursive_solution(step, f)
        } {
        forall(f: Nat -> U, g: Nat -> U) {
            if nat_recursive_solution_predicate(step, f) and
                nat_recursive_solution_predicate(step, g) {
                is_nat_recursive_solution(step, f)
                is_nat_recursive_solution(step, g)
                nat_recursive_solution_eq(step, f, g)
                f = g
            }
        }
        exists(f: Nat -> U) {
            nat_recursive_solution_predicate(step, f)
        }
        exists_unique_of_exists_and_pairwise(nat_recursive_solution_predicate(step))
        exists_unique(nat_recursive_solution_predicate(step))
    }
}

/// If a strict-natural-order recursive equation has a solution, then it is unique as a generic solution.
theorem nat_recursive_solution_generic_exists_unique[U](step: (Nat, Nat -> U) -> U) {
    is_predecessor_extensional_recursive_step(nat_lt_relation, step) and
    exists(f: Nat -> U) {
        is_nat_recursive_solution(step, f)
    } implies
    exists_unique(well_founded_recursive_solution_predicate(nat_lt_relation, step))
} by {
    if is_predecessor_extensional_recursive_step(nat_lt_relation, step) and
        exists(f: Nat -> U) {
            is_nat_recursive_solution(step, f)
        } {
        forall(f: Nat -> U, g: Nat -> U) {
            if well_founded_recursive_solution_predicate(nat_lt_relation, step, f) and
                well_founded_recursive_solution_predicate(nat_lt_relation, step, g) {
                is_well_founded_recursive_solution(nat_lt_relation, step, f)
                is_well_founded_recursive_solution(nat_lt_relation, step, g)
                is_nat_recursive_solution(step, f)
                is_nat_recursive_solution(step, g)
                nat_recursive_solution_eq(step, f, g)
                f = g
            }
        }
        let h: Nat -> U satisfy {
            is_nat_recursive_solution(step, h)
        }
        is_well_founded_recursive_solution(nat_lt_relation, step, h)
        well_founded_recursive_solution_predicate(nat_lt_relation, step, h)
        exists(k: Nat -> U) {
            well_founded_recursive_solution_predicate(nat_lt_relation, step, k)
        }
        exists_unique_of_exists_and_pairwise(well_founded_recursive_solution_predicate(nat_lt_relation, step))
        exists_unique(well_founded_recursive_solution_predicate(nat_lt_relation, step))
    }
}

/// The chosen strict-natural-order recursive solution satisfies the equation whenever a solution exists.
theorem choose_nat_recursive_solution_is_solution[U](step: (Nat, Nat -> U) -> U, default: U) {
    is_predecessor_extensional_recursive_step(nat_lt_relation, step) and
    exists(f: Nat -> U) {
        is_nat_recursive_solution(step, f)
    } implies
    is_nat_recursive_solution(step, choose_nat_recursive_solution(step, default))
} by {
    if is_predecessor_extensional_recursive_step(nat_lt_relation, step) and
        exists(f: Nat -> U) {
            is_nat_recursive_solution(step, f)
        } {
        nat_recursive_solution_generic_exists_unique(step)
        exists_unique(well_founded_recursive_solution_predicate(nat_lt_relation, step))
        exists(k: Nat -> U) {
            well_founded_recursive_solution_predicate(nat_lt_relation, step, k)
        }
        choose_or_default_spec(
            well_founded_recursive_solution_predicate(nat_lt_relation, step),
            well_founded_recursive_default_function[Nat, U](default)
        )
        well_founded_recursive_solution_predicate(
            nat_lt_relation,
            step,
            choose_well_founded_recursive_solution(nat_lt_relation, step, default)
        )
        well_founded_recursive_solution_predicate(nat_lt_relation, step, choose_nat_recursive_solution(step, default))
        is_well_founded_recursive_solution(nat_lt_relation, step, choose_nat_recursive_solution(step, default))
        is_nat_recursive_solution(step, choose_nat_recursive_solution(step, default))
    }
}

/// The chosen strict-natural-order recursive solution equals any supplied solution.
theorem choose_nat_recursive_solution_eq_solution[U](
    step: (Nat, Nat -> U) -> U,
    default: U,
    f: Nat -> U
) {
    is_predecessor_extensional_recursive_step(nat_lt_relation, step) and
    is_nat_recursive_solution(step, f) implies
    choose_nat_recursive_solution(step, default) = f
} by {
    if is_predecessor_extensional_recursive_step(nat_lt_relation, step) and
        is_nat_recursive_solution(step, f) {
        exists(g: Nat -> U) {
            is_nat_recursive_solution(step, g)
        }
        nat_recursive_solution_generic_exists_unique(step)
        exists_unique(well_founded_recursive_solution_predicate(nat_lt_relation, step))
        well_founded_recursive_solution_predicate(nat_lt_relation, step, f)
        choose_or_default_unique_eq(
            well_founded_recursive_solution_predicate(nat_lt_relation, step),
            well_founded_recursive_default_function[Nat, U](default),
            f
        )
        choose_nat_recursive_solution(step, default) = f
    }
}

/// The chosen strict-natural-order recursive solution does not depend on the default value when a solution exists.
theorem choose_nat_recursive_solution_default_independent[U](
    step: (Nat, Nat -> U) -> U,
    default1: U,
    default2: U
) {
    is_predecessor_extensional_recursive_step(nat_lt_relation, step) and
    exists(f: Nat -> U) {
        is_nat_recursive_solution(step, f)
    } implies
    choose_nat_recursive_solution(step, default1) = choose_nat_recursive_solution(step, default2)
} by {
    if is_predecessor_extensional_recursive_step(nat_lt_relation, step) and
        exists(f: Nat -> U) {
            is_nat_recursive_solution(step, f)
        } {
        nat_recursive_solution_generic_exists_unique(step)
        exists_unique(well_founded_recursive_solution_predicate(nat_lt_relation, step))
        choose_or_default_unique_default_eq(
            well_founded_recursive_solution_predicate(nat_lt_relation, step),
            well_founded_recursive_default_function[Nat, U](default1),
            well_founded_recursive_default_function[Nat, U](default2)
        )
        choose_nat_recursive_solution(step, default1) = choose_nat_recursive_solution(step, default2)
    }
}

/// The chosen strict-natural-order recursive solution satisfies the equation at a chosen number.
theorem choose_nat_recursive_solution_eq_step_at[U](
    step: (Nat, Nat -> U) -> U,
    default: U,
    n: Nat
) {
    is_predecessor_extensional_recursive_step(nat_lt_relation, step) and
    exists(f: Nat -> U) {
        is_nat_recursive_solution(step, f)
    } implies
    choose_nat_recursive_solution(step, default)(n) =
    step(n, choose_nat_recursive_solution(step, default))
} by {
    if is_predecessor_extensional_recursive_step(nat_lt_relation, step) and
        exists(f: Nat -> U) {
            is_nat_recursive_solution(step, f)
        } {
        choose_nat_recursive_solution_is_solution(step, default)
        is_nat_recursive_solution(step, choose_nat_recursive_solution(step, default))
        is_well_founded_recursive_solution(nat_lt_relation, step, choose_nat_recursive_solution(step, default)) =
            forall(x: Nat) {
                choose_nat_recursive_solution(step, default)(x) =
                step(x, choose_nat_recursive_solution(step, default))
            }
        choose_nat_recursive_solution(step, default)(n) =
        step(n, choose_nat_recursive_solution(step, default))
    }
}

/// Primitive recursion at zero is the base value.
theorem nat_primitive_recursive_function_zero[U](base: U, next: (Nat, U) -> U) {
    nat_primitive_recursive_function(base, next, Nat.0) = base
}

/// Primitive recursion at a successor is given by the successor step.
theorem nat_primitive_recursive_function_suc[U](base: U, next: (Nat, U) -> U, n: Nat) {
    nat_primitive_recursive_function(base, next, n.suc) =
    next(n, nat_primitive_recursive_function(base, next, n))
}

/// The primitive-recursion step at zero is the base value.
theorem nat_primitive_recursive_step_zero[U](base: U, next: (Nat, U) -> U, f: Nat -> U) {
    nat_primitive_recursive_step(base, next, Nat.0, f) = base
}

/// The primitive-recursion step at a successor uses the predecessor value.
theorem nat_primitive_recursive_step_suc[U](base: U, next: (Nat, U) -> U, f: Nat -> U, n: Nat) {
    nat_primitive_recursive_step(base, next, n.suc, f) = next(n, f(n))
}

/// The primitive-recursion step depends only on predecessor values.
theorem nat_primitive_recursive_step_is_predecessor_extensional[U](
    base: U,
    next: (Nat, U) -> U
) {
    is_predecessor_extensional_recursive_step(
        nat_lt_relation,
        nat_primitive_recursive_step(base, next)
    )
} by {
    forall(n: Nat, f: Nat -> U, g: Nat -> U) {
        match n {
            Nat.zero {
                if functions_agree_on_predecessors(nat_lt_relation, n, f, g) {
                    nat_primitive_recursive_step(base, next, n, f) = base
                    nat_primitive_recursive_step(base, next, n, g) = base
                    nat_primitive_recursive_step(base, next, n, f) =
                        nat_primitive_recursive_step(base, next, n, g)
                }
            }
            Nat.suc(pred) {
                if functions_agree_on_predecessors(nat_lt_relation, n, f, g) {
                    nat_lt_relation(pred, n)
                    functions_agree_on_predecessors(nat_lt_relation, n, f, g) =
                        forall(y: Nat) {
                            nat_lt_relation(y, n) implies f(y) = g(y)
                        }
                    f(pred) = g(pred)
                    nat_primitive_recursive_step(base, next, n, f) = next(pred, f(pred))
                    nat_primitive_recursive_step(base, next, n, g) = next(pred, g(pred))
                    nat_primitive_recursive_step(base, next, n, f) =
                        nat_primitive_recursive_step(base, next, n, g)
                }
            }
        }
    }
}

/// The primitive-recursive function satisfies its associated recursive equation.
theorem nat_primitive_recursive_function_is_solution[U](base: U, next: (Nat, U) -> U) {
    is_nat_recursive_solution(
        nat_primitive_recursive_step(base, next),
        nat_primitive_recursive_function(base, next)
    )
} by {
    is_nat_recursive_solution(
        nat_primitive_recursive_step(base, next),
        nat_primitive_recursive_function(base, next)
    ) =
        forall(n: Nat) {
            nat_primitive_recursive_function(base, next, n) =
            nat_primitive_recursive_step(
                base,
                next,
                n,
                nat_primitive_recursive_function(base, next)
            )
        }
    forall(n: Nat) {
        match n {
            Nat.zero {
                nat_primitive_recursive_function(base, next, n) = base
                nat_primitive_recursive_step(
                    base,
                    next,
                    n,
                    nat_primitive_recursive_function(base, next)
                ) = base
                nat_primitive_recursive_function(base, next, n) =
                nat_primitive_recursive_step(
                    base,
                    next,
                    n,
                    nat_primitive_recursive_function(base, next)
                )
            }
            Nat.suc(pred) {
                nat_primitive_recursive_function(base, next, n) =
                    next(pred, nat_primitive_recursive_function(base, next, pred))
                nat_primitive_recursive_step(
                    base,
                    next,
                    n,
                    nat_primitive_recursive_function(base, next)
                ) =
                    next(pred, nat_primitive_recursive_function(base, next, pred))
                nat_primitive_recursive_function(base, next, n) =
                nat_primitive_recursive_step(
                    base,
                    next,
                    n,
                    nat_primitive_recursive_function(base, next)
                )
            }
        }
    }
}

/// The primitive-recursive equation has a solution.
theorem nat_primitive_recursive_solution_exists[U](base: U, next: (Nat, U) -> U) {
    exists(f: Nat -> U) {
        is_nat_recursive_solution(nat_primitive_recursive_step(base, next), f)
    }
} by {
    nat_primitive_recursive_function_is_solution(base, next)
    exists(f: Nat -> U) {
        f = nat_primitive_recursive_function(base, next) and
        is_nat_recursive_solution(nat_primitive_recursive_step(base, next), f)
    }
}

/// The primitive-recursive equation has a solution as a well-founded recursive equation.
theorem nat_primitive_recursive_well_founded_solution_exists[U](
    base: U,
    next: (Nat, U) -> U
) {
    exists(f: Nat -> U) {
        is_well_founded_recursive_solution(
            nat_lt_relation,
            nat_primitive_recursive_step(base, next),
            f
        )
    }
} by {
    nat_primitive_recursive_function_is_solution(base, next)
    is_well_founded_recursive_solution(
        nat_lt_relation,
        nat_primitive_recursive_step(base, next),
        nat_primitive_recursive_function(base, next)
    )
    exists(f: Nat -> U) {
        f = nat_primitive_recursive_function(base, next) and
        is_well_founded_recursive_solution(
            nat_lt_relation,
            nat_primitive_recursive_step(base, next),
            f
        )
    }
}

/// The primitive-recursive equation has a unique solution.
theorem nat_primitive_recursive_solution_exists_unique[U](base: U, next: (Nat, U) -> U) {
    exists_unique(nat_recursive_solution_predicate(nat_primitive_recursive_step(base, next)))
} by {
    nat_primitive_recursive_step_is_predecessor_extensional(base, next)
    nat_primitive_recursive_solution_exists(base, next)
    nat_recursive_solution_exists_unique(nat_primitive_recursive_step(base, next))
    exists_unique(nat_recursive_solution_predicate(nat_primitive_recursive_step(base, next)))
}

/// The primitive-recursive equation has a unique well-founded recursive solution.
theorem nat_primitive_recursive_well_founded_solution_exists_unique[U](
    base: U,
    next: (Nat, U) -> U
) {
    exists_unique(
        well_founded_recursive_solution_predicate(
            nat_lt_relation,
            nat_primitive_recursive_step(base, next)
        )
    )
} by {
    nat_primitive_recursive_step_is_predecessor_extensional(base, next)
    nat_primitive_recursive_solution_exists(base, next)
    nat_recursive_solution_generic_exists_unique(nat_primitive_recursive_step(base, next))
    exists_unique(
        well_founded_recursive_solution_predicate(
            nat_lt_relation,
            nat_primitive_recursive_step(base, next)
        )
    )
}

/// Any solution of a primitive-recursive equation is the primitive-recursive function.
theorem nat_primitive_recursive_solution_eq_function[U](
    base: U,
    next: (Nat, U) -> U,
    f: Nat -> U
) {
    is_nat_recursive_solution(nat_primitive_recursive_step(base, next), f) implies
    f = nat_primitive_recursive_function(base, next)
} by {
    if is_nat_recursive_solution(nat_primitive_recursive_step(base, next), f) {
        nat_primitive_recursive_step_is_predecessor_extensional(base, next)
        nat_primitive_recursive_function_is_solution(base, next)
        nat_recursive_solution_eq(
            nat_primitive_recursive_step(base, next),
            f,
            nat_primitive_recursive_function(base, next)
        )
        f = nat_primitive_recursive_function(base, next)
    }
}

/// Any well-founded solution of a primitive-recursive equation is the primitive-recursive function.
theorem nat_primitive_recursive_well_founded_solution_eq_function[U](
    base: U,
    next: (Nat, U) -> U,
    f: Nat -> U
) {
    is_well_founded_recursive_solution(
        nat_lt_relation,
        nat_primitive_recursive_step(base, next),
        f
    ) implies
    f = nat_primitive_recursive_function(base, next)
} by {
    if is_well_founded_recursive_solution(
        nat_lt_relation,
        nat_primitive_recursive_step(base, next),
        f
    ) {
        is_nat_recursive_solution(nat_primitive_recursive_step(base, next), f)
        nat_primitive_recursive_solution_eq_function(base, next, f)
        f = nat_primitive_recursive_function(base, next)
    }
}

/// The chosen solution of a primitive-recursive equation is the primitive-recursive function.
theorem choose_nat_primitive_recursive_solution_eq_function[U](
    base: U,
    next: (Nat, U) -> U,
    default: U
) {
    choose_nat_recursive_solution(nat_primitive_recursive_step(base, next), default) =
    nat_primitive_recursive_function(base, next)
} by {
    nat_primitive_recursive_step_is_predecessor_extensional(base, next)
    nat_primitive_recursive_function_is_solution(base, next)
    choose_nat_recursive_solution_eq_solution(
        nat_primitive_recursive_step(base, next),
        default,
        nat_primitive_recursive_function(base, next)
    )
    choose_nat_recursive_solution(nat_primitive_recursive_step(base, next), default) =
    nat_primitive_recursive_function(base, next)
}

/// The chosen well-founded solution of a primitive-recursive equation is the primitive-recursive function.
theorem choose_well_founded_nat_primitive_recursive_solution_eq_function[U](
    base: U,
    next: (Nat, U) -> U,
    default: U
) {
    choose_well_founded_recursive_solution(
        nat_lt_relation,
        nat_primitive_recursive_step(base, next),
        default
    ) =
    nat_primitive_recursive_function(base, next)
} by {
    nat_primitive_recursive_step_is_predecessor_extensional(base, next)
    nat_primitive_recursive_function_is_solution(base, next)
    nat_primitive_recursive_well_founded_solution_exists_unique(base, next)
    choose_or_default_unique_eq(
        well_founded_recursive_solution_predicate(
            nat_lt_relation,
            nat_primitive_recursive_step(base, next)
        ),
        well_founded_recursive_default_function[Nat, U](default),
        nat_primitive_recursive_function(base, next)
    )
    choose_well_founded_recursive_solution(
        nat_lt_relation,
        nat_primitive_recursive_step(base, next),
        default
    ) =
    nat_primitive_recursive_function(base, next)
}

/// The chosen well-founded primitive-recursive solution is independent of the default value.
theorem choose_well_founded_nat_primitive_recursive_solution_default_independent[U](
    base: U,
    next: (Nat, U) -> U,
    default1: U,
    default2: U
) {
    choose_well_founded_recursive_solution(
        nat_lt_relation,
        nat_primitive_recursive_step(base, next),
        default1
    ) =
    choose_well_founded_recursive_solution(
        nat_lt_relation,
        nat_primitive_recursive_step(base, next),
        default2
    )
} by {
    choose_well_founded_nat_primitive_recursive_solution_eq_function(base, next, default1)
    choose_well_founded_nat_primitive_recursive_solution_eq_function(base, next, default2)
    choose_well_founded_recursive_solution(
        nat_lt_relation,
        nat_primitive_recursive_step(base, next),
        default1
    ) =
    choose_well_founded_recursive_solution(
        nat_lt_relation,
        nat_primitive_recursive_step(base, next),
        default2
    )
}

/// The chosen solution of a primitive-recursive equation satisfies the zero equation.
theorem choose_nat_primitive_recursive_solution_zero[U](
    base: U,
    next: (Nat, U) -> U,
    default: U
) {
    choose_nat_recursive_solution(nat_primitive_recursive_step(base, next), default)(Nat.0) = base
} by {
    choose_nat_primitive_recursive_solution_eq_function(base, next, default)
    nat_primitive_recursive_function_zero(base, next)
}

/// The chosen well-founded solution of a primitive-recursive equation has the base value at zero.
theorem choose_well_founded_nat_primitive_recursive_solution_zero[U](
    base: U,
    next: (Nat, U) -> U,
    default: U
) {
    choose_well_founded_recursive_solution(
        nat_lt_relation,
        nat_primitive_recursive_step(base, next),
        default
    )(Nat.0) = base
} by {
    choose_well_founded_nat_primitive_recursive_solution_eq_function(base, next, default)
    nat_primitive_recursive_function_zero(base, next)
}

/// The chosen solution of a primitive-recursive equation satisfies the successor equation.
theorem choose_nat_primitive_recursive_solution_suc[U](
    base: U,
    next: (Nat, U) -> U,
    default: U,
    n: Nat
) {
    choose_nat_recursive_solution(nat_primitive_recursive_step(base, next), default)(n.suc) =
    next(n, choose_nat_recursive_solution(nat_primitive_recursive_step(base, next), default)(n))
} by {
    choose_nat_primitive_recursive_solution_eq_function(base, next, default)
    nat_primitive_recursive_function_suc(base, next, n)
    choose_nat_recursive_solution(nat_primitive_recursive_step(base, next), default)(n.suc) =
        nat_primitive_recursive_function(base, next, n.suc)
    choose_nat_recursive_solution(nat_primitive_recursive_step(base, next), default)(n) =
        nat_primitive_recursive_function(base, next, n)
    choose_nat_recursive_solution(nat_primitive_recursive_step(base, next), default)(n.suc) =
    next(n, choose_nat_recursive_solution(nat_primitive_recursive_step(base, next), default)(n))
}

/// The chosen well-founded solution of a primitive-recursive equation has the successor value.
theorem choose_well_founded_nat_primitive_recursive_solution_suc[U](
    base: U,
    next: (Nat, U) -> U,
    default: U,
    n: Nat
) {
    choose_well_founded_recursive_solution(
        nat_lt_relation,
        nat_primitive_recursive_step(base, next),
        default
    )(n.suc) =
    next(
        n,
        choose_well_founded_recursive_solution(
            nat_lt_relation,
            nat_primitive_recursive_step(base, next),
            default
        )(n)
    )
} by {
    choose_well_founded_nat_primitive_recursive_solution_eq_function(base, next, default)
    nat_primitive_recursive_function_suc(base, next, n)
    choose_well_founded_recursive_solution(
        nat_lt_relation,
        nat_primitive_recursive_step(base, next),
        default
    )(n.suc) =
        nat_primitive_recursive_function(base, next, n.suc)
    choose_well_founded_recursive_solution(
        nat_lt_relation,
        nat_primitive_recursive_step(base, next),
        default
    )(n) =
        nat_primitive_recursive_function(base, next, n)
    choose_well_founded_recursive_solution(
        nat_lt_relation,
        nat_primitive_recursive_step(base, next),
        default
    )(n.suc) =
    next(
        n,
        choose_well_founded_recursive_solution(
            nat_lt_relation,
            nat_primitive_recursive_step(base, next),
            default
        )(n)
    )
}

/// The chosen primitive-recursive function is the chosen strict-natural-order solution.
theorem chosen_nat_primitive_recursive_function_eq_choose_solution[U](
    base: U,
    next: (Nat, U) -> U,
    default: U
) {
    chosen_nat_primitive_recursive_function(base, next, default) =
    choose_nat_recursive_solution(nat_primitive_recursive_step(base, next), default)
}

/// The chosen primitive-recursive function is the chosen well-founded solution.
theorem chosen_nat_primitive_recursive_function_eq_choose_well_founded_solution[U](
    base: U,
    next: (Nat, U) -> U,
    default: U
) {
    chosen_nat_primitive_recursive_function(base, next, default) =
    choose_well_founded_recursive_solution(
        nat_lt_relation,
        nat_primitive_recursive_step(base, next),
        default
    )
}

/// The chosen primitive-recursive function is the primitive-recursive function.
theorem chosen_nat_primitive_recursive_function_eq_function[U](
    base: U,
    next: (Nat, U) -> U,
    default: U
) {
    chosen_nat_primitive_recursive_function(base, next, default) =
    nat_primitive_recursive_function(base, next)
} by {
    choose_nat_primitive_recursive_solution_eq_function(base, next, default)
    chosen_nat_primitive_recursive_function(base, next, default) =
    choose_nat_recursive_solution(nat_primitive_recursive_step(base, next), default)
    chosen_nat_primitive_recursive_function(base, next, default) =
    nat_primitive_recursive_function(base, next)
}

/// The chosen primitive-recursive function satisfies its associated recursive equation.
theorem chosen_nat_primitive_recursive_function_is_solution[U](
    base: U,
    next: (Nat, U) -> U,
    default: U
) {
    is_nat_recursive_solution(
        nat_primitive_recursive_step(base, next),
        chosen_nat_primitive_recursive_function(base, next, default)
    )
} by {
    chosen_nat_primitive_recursive_function_eq_function(base, next, default)
    nat_primitive_recursive_function_is_solution(base, next)
    is_nat_recursive_solution(
        nat_primitive_recursive_step(base, next),
        chosen_nat_primitive_recursive_function(base, next, default)
    )
}

/// The chosen primitive-recursive function is a well-founded recursive solution.
theorem chosen_nat_primitive_recursive_function_is_well_founded_solution[U](
    base: U,
    next: (Nat, U) -> U,
    default: U
) {
    is_well_founded_recursive_solution(
        nat_lt_relation,
        nat_primitive_recursive_step(base, next),
        chosen_nat_primitive_recursive_function(base, next, default)
    )
} by {
    chosen_nat_primitive_recursive_function_is_solution(base, next, default)
    is_well_founded_recursive_solution(
        nat_lt_relation,
        nat_primitive_recursive_step(base, next),
        chosen_nat_primitive_recursive_function(base, next, default)
    )
}

/// The chosen primitive-recursive function is independent of the default value.
theorem chosen_nat_primitive_recursive_function_default_independent[U](
    base: U,
    next: (Nat, U) -> U,
    default1: U,
    default2: U
) {
    chosen_nat_primitive_recursive_function(base, next, default1) =
    chosen_nat_primitive_recursive_function(base, next, default2)
} by {
    chosen_nat_primitive_recursive_function_eq_function(base, next, default1)
    chosen_nat_primitive_recursive_function_eq_function(base, next, default2)
    chosen_nat_primitive_recursive_function(base, next, default1) =
    chosen_nat_primitive_recursive_function(base, next, default2)
}

/// The chosen primitive-recursive function has the base value at zero.
theorem chosen_nat_primitive_recursive_function_zero[U](
    base: U,
    next: (Nat, U) -> U,
    default: U
) {
    chosen_nat_primitive_recursive_function(base, next, default, Nat.0) = base
} by {
    chosen_nat_primitive_recursive_function_eq_function(base, next, default)
    nat_primitive_recursive_function_zero(base, next)
}

/// The chosen primitive-recursive function has the primitive-recursive successor value.
theorem chosen_nat_primitive_recursive_function_suc[U](
    base: U,
    next: (Nat, U) -> U,
    default: U,
    n: Nat
) {
    chosen_nat_primitive_recursive_function(base, next, default, n.suc) =
    next(n, chosen_nat_primitive_recursive_function(base, next, default, n))
} by {
    chosen_nat_primitive_recursive_function_eq_function(base, next, default)
    nat_primitive_recursive_function_suc(base, next, n)
    chosen_nat_primitive_recursive_function(base, next, default, n.suc) =
        nat_primitive_recursive_function(base, next, n.suc)
    chosen_nat_primitive_recursive_function(base, next, default, n) =
        nat_primitive_recursive_function(base, next, n)
    chosen_nat_primitive_recursive_function(base, next, default, n.suc) =
    next(n, chosen_nat_primitive_recursive_function(base, next, default, n))
}

/// The two-step recursion state at zero is the initial pair.
theorem nat_two_step_recursive_state_zero[U](initial: Pair[U, U], next: (Nat, U, U) -> U) {
    nat_two_step_recursive_state(initial, next, Nat.0) = initial
} by {
    nat_two_step_recursive_state(initial, next, Nat.0) =
        nat_primitive_recursive_function(initial, nat_two_step_recursive_state_next(next), Nat.0)
    nat_primitive_recursive_function_zero(initial, nat_two_step_recursive_state_next(next))
    nat_two_step_recursive_state(initial, next, Nat.0) = initial
}

/// The two-step recursion state advances by shifting the previous pair.
theorem nat_two_step_recursive_state_suc[U](initial: Pair[U, U], next: (Nat, U, U) -> U, n: Nat) {
    nat_two_step_recursive_state(initial, next, n.suc) =
    Pair.new(
        nat_two_step_recursive_state(initial, next, n).second,
        next(
            n,
            nat_two_step_recursive_state(initial, next, n).first,
            nat_two_step_recursive_state(initial, next, n).second
        )
    )
} by {
    nat_two_step_recursive_state(initial, next, n.suc) =
        nat_primitive_recursive_function(initial, nat_two_step_recursive_state_next(next), n.suc)
    nat_primitive_recursive_function_suc(initial, nat_two_step_recursive_state_next(next), n)
    nat_two_step_recursive_state_next(next, n, nat_two_step_recursive_state(initial, next, n)) =
    Pair.new(
        nat_two_step_recursive_state(initial, next, n).second,
        next(
            n,
            nat_two_step_recursive_state(initial, next, n).first,
            nat_two_step_recursive_state(initial, next, n).second
        )
    )
    nat_two_step_recursive_state(initial, next, n.suc) =
    Pair.new(
        nat_two_step_recursive_state(initial, next, n).second,
        next(
            n,
            nat_two_step_recursive_state(initial, next, n).first,
            nat_two_step_recursive_state(initial, next, n).second
        )
    )
}

/// The two-step recursive function has the first initial value at zero.
theorem nat_two_step_recursive_function_zero[U](initial: Pair[U, U], next: (Nat, U, U) -> U) {
    nat_two_step_recursive_function(initial, next, Nat.0) = initial.first
} by {
    nat_two_step_recursive_function(initial, next, Nat.0) =
        nat_two_step_recursive_state(initial, next, Nat.0).first
    nat_two_step_recursive_state_zero(initial, next)
    nat_two_step_recursive_function(initial, next, Nat.0) = initial.first
}

/// The two-step recursive function has the second initial value at one.
theorem nat_two_step_recursive_function_one[U](initial: Pair[U, U], next: (Nat, U, U) -> U) {
    nat_two_step_recursive_function(initial, next, Nat.1) = initial.second
} by {
    nat_two_step_recursive_function(initial, next, Nat.1) =
        nat_two_step_recursive_state(initial, next, Nat.1).first
    nat_two_step_recursive_state_suc(initial, next, Nat.0)
    nat_two_step_recursive_state(initial, next, Nat.1) =
    Pair.new(
        nat_two_step_recursive_state(initial, next, Nat.0).second,
        next(
            Nat.0,
            nat_two_step_recursive_state(initial, next, Nat.0).first,
            nat_two_step_recursive_state(initial, next, Nat.0).second
        )
    )
    nat_two_step_recursive_state_zero(initial, next)
    pair_new_first(initial.second, next(Nat.0, initial.first, initial.second))
    nat_two_step_recursive_state(initial, next, Nat.1).first = initial.second
    nat_two_step_recursive_function(initial, next, Nat.1) = initial.second
}

/// The second projection of the state is the next value of the two-step recursive function.
theorem nat_two_step_recursive_state_second_eq_function_suc[U](
    initial: Pair[U, U],
    next: (Nat, U, U) -> U,
    n: Nat
) {
    nat_two_step_recursive_state(initial, next, n).second =
    nat_two_step_recursive_function(initial, next, n.suc)
} by {
    nat_two_step_recursive_function(initial, next, n.suc) =
        nat_two_step_recursive_state(initial, next, n.suc).first
    nat_two_step_recursive_state_suc(initial, next, n)
    nat_two_step_recursive_state(initial, next, n.suc).first =
        nat_two_step_recursive_state(initial, next, n).second
    nat_two_step_recursive_state(initial, next, n).second =
    nat_two_step_recursive_function(initial, next, n.suc)
}

/// The two-step recursive function satisfies the successor-successor equation.
theorem nat_two_step_recursive_function_suc_suc[U](
    initial: Pair[U, U],
    next: (Nat, U, U) -> U,
    n: Nat
) {
    nat_two_step_recursive_function(initial, next, n.suc.suc) =
    next(
        n,
        nat_two_step_recursive_function(initial, next, n),
        nat_two_step_recursive_function(initial, next, n.suc)
    )
} by {
    nat_two_step_recursive_function(initial, next, n.suc.suc) =
        nat_two_step_recursive_state(initial, next, n.suc.suc).first
    nat_two_step_recursive_state_suc(initial, next, n.suc)
    nat_two_step_recursive_state(initial, next, n.suc.suc).first =
        nat_two_step_recursive_state(initial, next, n.suc).second
    nat_two_step_recursive_state_suc(initial, next, n)
    nat_two_step_recursive_state(initial, next, n.suc).second =
        next(
            n,
            nat_two_step_recursive_state(initial, next, n).first,
            nat_two_step_recursive_state(initial, next, n).second
        )
    nat_two_step_recursive_state(initial, next, n).first =
        nat_two_step_recursive_function(initial, next, n)
    nat_two_step_recursive_state_second_eq_function_suc(initial, next, n)
    nat_two_step_recursive_function(initial, next, n.suc.suc) =
    next(
        n,
        nat_two_step_recursive_function(initial, next, n),
        nat_two_step_recursive_function(initial, next, n.suc)
    )
}

/// The two-step-recursion step at zero is the first initial value.
theorem nat_two_step_recursive_step_zero[U](initial: Pair[U, U], next: (Nat, U, U) -> U, f: Nat -> U) {
    nat_two_step_recursive_step(initial, next, Nat.0, f) = initial.first
}

/// The two-step-recursion step at one is the second initial value.
theorem nat_two_step_recursive_step_one[U](initial: Pair[U, U], next: (Nat, U, U) -> U, f: Nat -> U) {
    nat_two_step_recursive_step(initial, next, Nat.1, f) = initial.second
}

/// The two-step-recursion step at a successor-successor uses the two preceding values.
theorem nat_two_step_recursive_step_suc_suc[U](
    initial: Pair[U, U],
    next: (Nat, U, U) -> U,
    f: Nat -> U,
    n: Nat
) {
    nat_two_step_recursive_step(initial, next, n.suc.suc, f) = next(n, f(n), f(n.suc))
}

/// The two-step-recursion step depends only on predecessor values.
theorem nat_two_step_recursive_step_is_predecessor_extensional[U](
    initial: Pair[U, U],
    next: (Nat, U, U) -> U
) {
    is_predecessor_extensional_recursive_step(
        nat_lt_relation,
        nat_two_step_recursive_step(initial, next)
    )
} by {
    forall(n: Nat, f: Nat -> U, g: Nat -> U) {
        match n {
            Nat.zero {
                if functions_agree_on_predecessors(nat_lt_relation, n, f, g) {
                    nat_two_step_recursive_step(initial, next, n, f) = initial.first
                    nat_two_step_recursive_step(initial, next, n, g) = initial.first
                    nat_two_step_recursive_step(initial, next, n, f) =
                        nat_two_step_recursive_step(initial, next, n, g)
                }
            }
            Nat.suc(pred) {
                match pred {
                    Nat.zero {
                        if functions_agree_on_predecessors(nat_lt_relation, n, f, g) {
                            nat_two_step_recursive_step(initial, next, n, f) = initial.second
                            nat_two_step_recursive_step(initial, next, n, g) = initial.second
                            nat_two_step_recursive_step(initial, next, n, f) =
                                nat_two_step_recursive_step(initial, next, n, g)
                        }
                    }
                    Nat.suc(prev) {
                        if functions_agree_on_predecessors(nat_lt_relation, n, f, g) {
                            lt_suc(prev)
                            lt_suc(pred)
                            lt_trans(prev, pred, n)
                            nat_lt_relation(prev, n)
                            nat_lt_relation(pred, n)
                            functions_agree_on_predecessors(nat_lt_relation, n, f, g) =
                                forall(y: Nat) {
                                    nat_lt_relation(y, n) implies f(y) = g(y)
                                }
                            f(prev) = g(prev)
                            f(pred) = g(pred)
                            nat_two_step_recursive_step(initial, next, n, f) =
                                next(prev, f(prev), f(pred))
                            nat_two_step_recursive_step(initial, next, n, g) =
                                next(prev, g(prev), g(pred))
                            nat_two_step_recursive_step(initial, next, n, f) =
                                nat_two_step_recursive_step(initial, next, n, g)
                        }
                    }
                }
            }
        }
    }
}

/// The two-step recursive function satisfies its associated recursive equation.
theorem nat_two_step_recursive_function_is_solution[U](initial: Pair[U, U], next: (Nat, U, U) -> U) {
    is_nat_recursive_solution(
        nat_two_step_recursive_step(initial, next),
        nat_two_step_recursive_function(initial, next)
    )
} by {
    is_nat_recursive_solution(
        nat_two_step_recursive_step(initial, next),
        nat_two_step_recursive_function(initial, next)
    ) =
        forall(n: Nat) {
            nat_two_step_recursive_function(initial, next, n) =
            nat_two_step_recursive_step(
                initial,
                next,
                n,
                nat_two_step_recursive_function(initial, next)
            )
        }
    forall(n: Nat) {
        match n {
            Nat.zero {
                nat_two_step_recursive_function_zero(initial, next)
                nat_two_step_recursive_step_zero(initial, next, nat_two_step_recursive_function(initial, next))
                nat_two_step_recursive_function(initial, next, n) =
                nat_two_step_recursive_step(
                    initial,
                    next,
                    n,
                    nat_two_step_recursive_function(initial, next)
                )
            }
            Nat.suc(pred) {
                match pred {
                    Nat.zero {
                        nat_two_step_recursive_function_one(initial, next)
                        nat_two_step_recursive_step_one(initial, next, nat_two_step_recursive_function(initial, next))
                        nat_two_step_recursive_function(initial, next, n) =
                        nat_two_step_recursive_step(
                            initial,
                            next,
                            n,
                            nat_two_step_recursive_function(initial, next)
                        )
                    }
                    Nat.suc(prev) {
                        nat_two_step_recursive_function_suc_suc(initial, next, prev)
                        nat_two_step_recursive_step_suc_suc(
                            initial,
                            next,
                            nat_two_step_recursive_function(initial, next),
                            prev
                        )
                        nat_two_step_recursive_function(initial, next, n) =
                        nat_two_step_recursive_step(
                            initial,
                            next,
                            n,
                            nat_two_step_recursive_function(initial, next)
                        )
                    }
                }
            }
        }
    }
}

/// The two-step-recursive equation has a solution.
theorem nat_two_step_recursive_solution_exists[U](initial: Pair[U, U], next: (Nat, U, U) -> U) {
    exists(f: Nat -> U) {
        is_nat_recursive_solution(nat_two_step_recursive_step(initial, next), f)
    }
} by {
    nat_two_step_recursive_function_is_solution(initial, next)
    exists(f: Nat -> U) {
        f = nat_two_step_recursive_function(initial, next) and
        is_nat_recursive_solution(nat_two_step_recursive_step(initial, next), f)
    }
}

/// The two-step-recursive equation has a unique solution.
theorem nat_two_step_recursive_solution_exists_unique[U](initial: Pair[U, U], next: (Nat, U, U) -> U) {
    exists_unique(nat_recursive_solution_predicate(nat_two_step_recursive_step(initial, next)))
} by {
    nat_two_step_recursive_step_is_predecessor_extensional(initial, next)
    nat_two_step_recursive_solution_exists(initial, next)
    nat_recursive_solution_exists_unique(nat_two_step_recursive_step(initial, next))
    exists_unique(nat_recursive_solution_predicate(nat_two_step_recursive_step(initial, next)))
}

/// Any solution of a two-step-recursive equation is the two-step recursive function.
theorem nat_two_step_recursive_solution_eq_function[U](
    initial: Pair[U, U],
    next: (Nat, U, U) -> U,
    f: Nat -> U
) {
    is_nat_recursive_solution(nat_two_step_recursive_step(initial, next), f) implies
    f = nat_two_step_recursive_function(initial, next)
} by {
    if is_nat_recursive_solution(nat_two_step_recursive_step(initial, next), f) {
        nat_two_step_recursive_step_is_predecessor_extensional(initial, next)
        nat_two_step_recursive_function_is_solution(initial, next)
        nat_recursive_solution_eq(
            nat_two_step_recursive_step(initial, next),
            f,
            nat_two_step_recursive_function(initial, next)
        )
        f = nat_two_step_recursive_function(initial, next)
    }
}

/// The chosen solution of a two-step-recursive equation is the two-step recursive function.
theorem choose_nat_two_step_recursive_solution_eq_function[U](
    initial: Pair[U, U],
    next: (Nat, U, U) -> U,
    default: U
) {
    choose_nat_recursive_solution(nat_two_step_recursive_step(initial, next), default) =
    nat_two_step_recursive_function(initial, next)
} by {
    nat_two_step_recursive_step_is_predecessor_extensional(initial, next)
    nat_two_step_recursive_function_is_solution(initial, next)
    choose_nat_recursive_solution_eq_solution(
        nat_two_step_recursive_step(initial, next),
        default,
        nat_two_step_recursive_function(initial, next)
    )
    choose_nat_recursive_solution(nat_two_step_recursive_step(initial, next), default) =
    nat_two_step_recursive_function(initial, next)
}

/// The chosen two-step-recursive function is the two-step recursive function.
theorem chosen_nat_two_step_recursive_function_eq_function[U](
    initial: Pair[U, U],
    next: (Nat, U, U) -> U,
    default: U
) {
    chosen_nat_two_step_recursive_function(initial, next, default) =
    nat_two_step_recursive_function(initial, next)
} by {
    choose_nat_two_step_recursive_solution_eq_function(initial, next, default)
    chosen_nat_two_step_recursive_function(initial, next, default) =
    choose_nat_recursive_solution(nat_two_step_recursive_step(initial, next), default)
    chosen_nat_two_step_recursive_function(initial, next, default) =
    nat_two_step_recursive_function(initial, next)
}

/// The chosen two-step-recursive function has the first initial value at zero.
theorem chosen_nat_two_step_recursive_function_zero[U](
    initial: Pair[U, U],
    next: (Nat, U, U) -> U,
    default: U
) {
    chosen_nat_two_step_recursive_function(initial, next, default, Nat.0) = initial.first
} by {
    chosen_nat_two_step_recursive_function_eq_function(initial, next, default)
    nat_two_step_recursive_function_zero(initial, next)
}

/// The chosen two-step-recursive function has the second initial value at one.
theorem chosen_nat_two_step_recursive_function_one[U](
    initial: Pair[U, U],
    next: (Nat, U, U) -> U,
    default: U
) {
    chosen_nat_two_step_recursive_function(initial, next, default, Nat.1) = initial.second
} by {
    chosen_nat_two_step_recursive_function_eq_function(initial, next, default)
    nat_two_step_recursive_function_one(initial, next)
}

/// The chosen two-step-recursive function satisfies the successor-successor equation.
theorem chosen_nat_two_step_recursive_function_suc_suc[U](
    initial: Pair[U, U],
    next: (Nat, U, U) -> U,
    default: U,
    n: Nat
) {
    chosen_nat_two_step_recursive_function(initial, next, default, n.suc.suc) =
    next(
        n,
        chosen_nat_two_step_recursive_function(initial, next, default, n),
        chosen_nat_two_step_recursive_function(initial, next, default, n.suc)
    )
} by {
    chosen_nat_two_step_recursive_function_eq_function(initial, next, default)
    nat_two_step_recursive_function_suc_suc(initial, next, n)
    chosen_nat_two_step_recursive_function(initial, next, default, n.suc.suc) =
        nat_two_step_recursive_function(initial, next, n.suc.suc)
    chosen_nat_two_step_recursive_function(initial, next, default, n) =
        nat_two_step_recursive_function(initial, next, n)
    chosen_nat_two_step_recursive_function(initial, next, default, n.suc) =
        nat_two_step_recursive_function(initial, next, n.suc)
    chosen_nat_two_step_recursive_function(initial, next, default, n.suc.suc) =
    next(
        n,
        chosen_nat_two_step_recursive_function(initial, next, default, n),
        chosen_nat_two_step_recursive_function(initial, next, default, n.suc)
    )
}


/// A well-founded relation has no infinite descending chain.
theorem well_founded_has_no_descending_chain[T](r: (T, T) -> Bool, c: Nat -> T) {
    is_well_founded(r) implies not is_descending_chain(r, c)
} by {
    if is_well_founded(r) {
        is_well_founded(r) = forall(p: T -> Bool) {
            (exists(x: T) { p(x) }) implies exists(m: T) {
                p(m) and forall(y: T) {
                    r(y, m) implies not p(y)
                }
            }
        }
        let p: T -> Bool = function(x: T) {
            exists(n: Nat) {
                c(n) = x
            }
        }
        exists(x: T) {
            p(x)
        }
        exists(m: T) {
            p(m) and forall(y: T) {
                r(y, m) implies not p(y)
            }
        }
        let m: T satisfy {
            p(m) and forall(y: T) {
                r(y, m) implies not p(y)
            }
        }
        let k: Nat satisfy {
            c(k) = m
        }
        forall(y: T) {
            r(y, m) implies not p(y)
        }
        if is_descending_chain(r, c) {
            is_descending_chain(r, c) = forall(n: Nat) {
                r(c(n.suc), c(n))
            }
            r(c(k.suc), c(k))
            r(c(k.suc), m)
            not p(c(k.suc))
            exists(n: Nat) {
                c(n) = c(k.suc)
            }
            p(c(k.suc))
            false
        }
        not is_descending_chain(r, c)
    }
}

/// A well-founded relation has no infinite descending chain.
theorem well_founded_has_no_descending_chain_exists[T](r: (T, T) -> Bool) {
    is_well_founded(r) implies has_no_descending_chain(r)
} by {
    if is_well_founded(r) {
        if has_descending_chain(r) {
            let c: Nat -> T satisfy {
                is_descending_chain(r, c)
            }
            well_founded_has_no_descending_chain(r, c)
            not is_descending_chain(r, c)
            false
        }
        not has_descending_chain(r)
        has_no_descending_chain(r)
    }
}

/// A Noetherian relation has no infinite ascending chain.
theorem noetherian_relation_has_no_ascending_chain[T](r: (T, T) -> Bool) {
    is_noetherian_relation(r) implies has_no_ascending_chain(r)
} by {
    if is_noetherian_relation(r) {
        is_well_founded(relation_converse(r))
        well_founded_has_no_descending_chain_exists(relation_converse(r))
        has_no_descending_chain(relation_converse(r))
        if has_ascending_chain(r) {
            has_ascending_chain_iff_has_descending_chain_converse(r)
            has_descending_chain(relation_converse(r))
            not has_descending_chain(relation_converse(r))
            false
        }
        not has_ascending_chain(r)
        has_no_ascending_chain(r)
    }
}

/// An infinite ascending chain contradicts Noetherianity.
theorem ascending_chain_not_noetherian_relation[T](r: (T, T) -> Bool, c: Nat -> T) {
    is_ascending_chain(r, c) implies not is_noetherian_relation(r)
} by {
    if is_ascending_chain(r, c) {
        if is_noetherian_relation(r) {
            noetherian_relation_has_no_ascending_chain(r)
            has_no_ascending_chain(r)
            exists(d: Nat -> T) {
                d = c and is_ascending_chain(r, d)
            }
            has_ascending_chain(r)
            not has_ascending_chain(r)
            false
        }
        not is_noetherian_relation(r)
    }
}

/// Existence of an infinite ascending chain contradicts Noetherianity.
theorem has_ascending_chain_not_noetherian_relation[T](r: (T, T) -> Bool) {
    has_ascending_chain(r) implies not is_noetherian_relation(r)
} by {
    if has_ascending_chain(r) {
        let c: Nat -> T satisfy {
            is_ascending_chain(r, c)
        }
        ascending_chain_not_noetherian_relation(r, c)
        not is_noetherian_relation(r)
    }
}

/// Relation inclusion transports ascending chains forward.
theorem relation_subset_ascending_chain[T](r: (T, T) -> Bool, s: (T, T) -> Bool, c: Nat -> T) {
    relation_subset(r, s) and is_ascending_chain(r, c) implies is_ascending_chain(s, c)
} by {
    if relation_subset(r, s) and is_ascending_chain(r, c) {
        is_ascending_chain(r, c) = forall(n: Nat) {
            r(c(n), c(n.suc))
        }
        forall(n: Nat) {
            r(c(n), c(n.suc))
            relation_subset_step(r, s, c(n), c(n.suc))
            s(c(n), c(n.suc))
        }
        is_ascending_chain(s, c)
    }
}

/// Relation inclusion transports existence of ascending chains forward.
theorem relation_subset_has_ascending_chain[T](r: (T, T) -> Bool, s: (T, T) -> Bool) {
    relation_subset(r, s) and has_ascending_chain(r) implies has_ascending_chain(s)
} by {
    if relation_subset(r, s) and has_ascending_chain(r) {
        let c: Nat -> T satisfy {
            is_ascending_chain(r, c)
        }
        relation_subset_ascending_chain(r, s, c)
        is_ascending_chain(s, c)
        exists(d: Nat -> T) {
            is_ascending_chain(s, d)
        }
        has_ascending_chain(s)
    }
}

/// Absence of ascending chains is inherited by subrelations.
theorem relation_subset_has_no_ascending_chain[T](r: (T, T) -> Bool, s: (T, T) -> Bool) {
    relation_subset(r, s) and has_no_ascending_chain(s) implies has_no_ascending_chain(r)
} by {
    if relation_subset(r, s) and has_no_ascending_chain(s) {
        if has_ascending_chain(r) {
            relation_subset_has_ascending_chain(r, s)
            has_ascending_chain(s)
            not has_ascending_chain(s)
            false
        }
        not has_ascending_chain(r)
        has_no_ascending_chain(r)
    }
}

/// The intersection of a relation with a relation that has no ascending chain also has no ascending chain.
theorem relation_intersection_left_has_no_ascending_chain[T](r: (T, T) -> Bool, s: (T, T) -> Bool) {
    has_no_ascending_chain(r) implies has_no_ascending_chain(relation_intersection(r, s))
} by {
    if has_no_ascending_chain(r) {
        relation_intersection_left_subset(r, s)
        relation_subset_has_no_ascending_chain(relation_intersection(r, s), r)
        has_no_ascending_chain(relation_intersection(r, s))
    }
}

/// The intersection of any relation with a relation that has no ascending chain also has no ascending chain.
theorem relation_intersection_right_has_no_ascending_chain[T](r: (T, T) -> Bool, s: (T, T) -> Bool) {
    has_no_ascending_chain(s) implies has_no_ascending_chain(relation_intersection(r, s))
} by {
    if has_no_ascending_chain(s) {
        relation_intersection_right_subset(r, s)
        relation_subset_has_no_ascending_chain(relation_intersection(r, s), s)
        has_no_ascending_chain(relation_intersection(r, s))
    }
}

/// Absence of ascending chains is invariant under mutual relation inclusion.
theorem relation_subset_both_has_no_ascending_chain[T](r: (T, T) -> Bool, s: (T, T) -> Bool) {
    relation_subset(r, s) and relation_subset(s, r) and has_no_ascending_chain(s) implies
    has_no_ascending_chain(r)
} by {
    if relation_subset(r, s) and relation_subset(s, r) and has_no_ascending_chain(s) {
        relation_subset_has_no_ascending_chain(r, s)
        has_no_ascending_chain(r)
    }
}

/// The image of a pullback ascending chain is an ascending chain.
theorem relation_pullback_ascending_chain_map_is_ascending[A, B](
    f: A -> B,
    r: (B, B) -> Bool,
    c: Nat -> A
) {
    is_ascending_chain(relation_pullback(f, r), c) implies
    is_ascending_chain(r, descending_chain_map(f, c))
} by {
    if is_ascending_chain(relation_pullback(f, r), c) {
        is_ascending_chain(relation_pullback(f, r), c) = forall(n: Nat) {
            relation_pullback(f, r, c(n), c(n.suc))
        }
        forall(n: Nat) {
            relation_pullback(f, r, c(n), c(n.suc))
            r(f(c(n)), f(c(n.suc)))
            descending_chain_map(f, c, n) = f(c(n))
            descending_chain_map(f, c, n.suc) = f(c(n.suc))
            r(descending_chain_map(f, c, n), descending_chain_map(f, c, n.suc))
        }
        is_ascending_chain(r, descending_chain_map(f, c))
    }
}

/// Pullbacks inherit absence of ascending chains.
theorem relation_pullback_has_no_ascending_chain[A, B](f: A -> B, r: (B, B) -> Bool) {
    has_no_ascending_chain(r) implies has_no_ascending_chain(relation_pullback(f, r))
} by {
    if has_no_ascending_chain(r) {
        if has_ascending_chain(relation_pullback(f, r)) {
            let c: Nat -> A satisfy {
                is_ascending_chain(relation_pullback(f, r), c)
            }
            relation_pullback_ascending_chain_map_is_ascending(f, r, c)
            is_ascending_chain(r, descending_chain_map(f, c))
            has_ascending_chain(r)
            not has_ascending_chain(r)
            false
        }
        not has_ascending_chain(relation_pullback(f, r))
        has_no_ascending_chain(relation_pullback(f, r))
    }
}

/// A pullback ascending chain gives an ascending chain in the target relation.
theorem relation_pullback_has_ascending_chain_imp[A, B](f: A -> B, r: (B, B) -> Bool) {
    has_ascending_chain(relation_pullback(f, r)) implies has_ascending_chain(r)
} by {
    if has_ascending_chain(relation_pullback(f, r)) {
        let c: Nat -> A satisfy {
            is_ascending_chain(relation_pullback(f, r), c)
        }
        relation_pullback_ascending_chain_map_is_ascending(f, r, c)
        is_ascending_chain(r, descending_chain_map(f, c))
        exists(d: Nat -> B) {
            is_ascending_chain(r, d)
        }
        has_ascending_chain(r)
    }
}

/// Relations controlled by a pullback inherit absence of ascending chains.
theorem relation_subset_pullback_has_no_ascending_chain[A, B](
    f: A -> B,
    r: (A, A) -> Bool,
    s: (B, B) -> Bool
) {
    relation_subset(r, relation_pullback(f, s)) and has_no_ascending_chain(s) implies
    has_no_ascending_chain(r)
} by {
    if relation_subset(r, relation_pullback(f, s)) and has_no_ascending_chain(s) {
        relation_pullback_has_no_ascending_chain(f, s)
        has_no_ascending_chain(relation_pullback(f, s))
        relation_subset_has_no_ascending_chain(r, relation_pullback(f, s))
        has_no_ascending_chain(r)
    }
}

/// An ascending chain in a relation controlled by a pullback gives an ascending chain in the target.
theorem relation_subset_pullback_has_ascending_chain[A, B](
    f: A -> B,
    r: (A, A) -> Bool,
    s: (B, B) -> Bool
) {
    relation_subset(r, relation_pullback(f, s)) and has_ascending_chain(r) implies
    has_ascending_chain(s)
} by {
    if relation_subset(r, relation_pullback(f, s)) and has_ascending_chain(r) {
        relation_subset_has_ascending_chain(r, relation_pullback(f, s))
        has_ascending_chain(relation_pullback(f, s))
        relation_pullback_has_ascending_chain_imp(f, s)
        has_ascending_chain(s)
    }
}

/// No sequence of naturals descends strictly at every successor step.
theorem nat_lt_relation_not_descending_chain(c: Nat -> Nat) {
    not is_descending_chain(nat_lt_relation, c)
} by {
    let p: Nat -> Bool = function(x: Nat) {
        exists(n: Nat) {
            c(n) = x
        }
    }
    exists(x: Nat) {
        p(x)
    }
    nat_lt_relation_has_minimal(p)
    exists(m: Nat) {
        p(m) and forall(y: Nat) {
            nat_lt_relation(y, m) implies not p(y)
        }
    }
    let m: Nat satisfy {
        p(m) and forall(y: Nat) {
            nat_lt_relation(y, m) implies not p(y)
        }
    }
    let k: Nat satisfy {
        c(k) = m
    }
    forall(y: Nat) {
        nat_lt_relation(y, m) implies not p(y)
    }
    if is_descending_chain(nat_lt_relation, c) {
        is_descending_chain(nat_lt_relation, c) = forall(n: Nat) {
            nat_lt_relation(c(n.suc), c(n))
        }
        nat_lt_relation(c(k.suc), c(k))
        nat_lt_relation(c(k.suc), m)
        not p(c(k.suc))
        exists(n: Nat) {
            c(n) = c(k.suc)
        }
        p(c(k.suc))
        false
    }
    not is_descending_chain(nat_lt_relation, c)
}

/// The strict natural order has no infinite descending chain.
theorem nat_lt_relation_has_no_descending_chain {
    has_no_descending_chain(nat_lt_relation)
} by {
    if has_descending_chain(nat_lt_relation) {
        let c: Nat -> Nat satisfy {
            is_descending_chain(nat_lt_relation, c)
        }
        nat_lt_relation_not_descending_chain(c)
        not is_descending_chain(nat_lt_relation, c)
        false
    }
    not has_descending_chain(nat_lt_relation)
    has_no_descending_chain(nat_lt_relation)
}

/// A bijective pushforward of a well-founded relation has no infinite descending chain.
theorem relation_pushforward_has_no_descending_chain_of_bijection[A, B](f: A -> B, r: (A, A) -> Bool) {
    is_bijection_fn(f) and is_well_founded(r) implies has_no_descending_chain(relation_pushforward(f, r))
} by {
    if is_bijection_fn(f) and is_well_founded(r) {
        relation_pushforward_is_well_founded_of_bijection(f, r)
        is_well_founded(relation_pushforward(f, r))
        well_founded_has_no_descending_chain_exists(relation_pushforward(f, r))
        has_no_descending_chain(relation_pushforward(f, r))
    }
}

/// An injective pushforward of a well-founded relation has no infinite descending chain.
theorem relation_pushforward_has_no_descending_chain_of_injection[A, B](f: A -> B, r: (A, A) -> Bool) {
    is_injective_fn(f) and is_well_founded(r) implies has_no_descending_chain(relation_pushforward(f, r))
} by {
    if is_injective_fn(f) and is_well_founded(r) {
        relation_pushforward_is_well_founded_of_injection(f, r)
        is_well_founded(relation_pushforward(f, r))
        well_founded_has_no_descending_chain_exists(relation_pushforward(f, r))
        has_no_descending_chain(relation_pushforward(f, r))
    }
}

/// A bundled bijective pushforward of a well-founded relation has no infinite descending chain.
theorem relation_pushforward_has_no_descending_chain_of_bijection_map[A, B](e: Bijection[A, B], r: (A, A) -> Bool) {
    is_well_founded(r) implies has_no_descending_chain(relation_pushforward(e.map, r))
} by {
    if is_well_founded(r) {
        relation_pushforward_is_well_founded_of_bijection_map(e, r)
        is_well_founded(relation_pushforward(e.map, r))
        well_founded_has_no_descending_chain_exists(relation_pushforward(e.map, r))
        has_no_descending_chain(relation_pushforward(e.map, r))
    }
}

/// An infinite descending chain contradicts well-foundedness.
theorem descending_chain_not_well_founded[T](r: (T, T) -> Bool, c: Nat -> T) {
    is_descending_chain(r, c) implies not is_well_founded(r)
} by {
    if is_descending_chain(r, c) {
        if is_well_founded(r) {
            well_founded_has_no_descending_chain(r, c)
            not is_descending_chain(r, c)
            false
        }
        not is_well_founded(r)
    }
}

/// The existence of an infinite descending chain contradicts well-foundedness.
theorem has_descending_chain_not_well_founded[T](r: (T, T) -> Bool) {
    has_descending_chain(r) implies not is_well_founded(r)
} by {
    if has_descending_chain(r) {
        let c: Nat -> T satisfy {
            is_descending_chain(r, c)
        }
        descending_chain_not_well_founded(r, c)
        not is_well_founded(r)
    }
}

/// Relation inclusion transports descending chains forward.
theorem relation_subset_descending_chain[T](r: (T, T) -> Bool, s: (T, T) -> Bool, c: Nat -> T) {
    relation_subset(r, s) and is_descending_chain(r, c) implies is_descending_chain(s, c)
} by {
    if relation_subset(r, s) and is_descending_chain(r, c) {
        is_descending_chain(r, c) = forall(n: Nat) {
            r(c(n.suc), c(n))
        }
        forall(n: Nat) {
            r(c(n.suc), c(n))
            relation_subset_step(r, s, c(n.suc), c(n))
            s(c(n.suc), c(n))
        }
        is_descending_chain(s, c)
    }
}

/// Relation inclusion transports existence of descending chains forward.
theorem relation_subset_has_descending_chain[T](r: (T, T) -> Bool, s: (T, T) -> Bool) {
    relation_subset(r, s) and has_descending_chain(r) implies has_descending_chain(s)
} by {
    if relation_subset(r, s) and has_descending_chain(r) {
        let c: Nat -> T satisfy {
            is_descending_chain(r, c)
        }
        relation_subset_descending_chain(r, s, c)
        is_descending_chain(s, c)
        exists(d: Nat -> T) {
            is_descending_chain(s, d)
        }
        has_descending_chain(s)
    }
}

/// Absence of descending chains is inherited by subrelations.
theorem relation_subset_has_no_descending_chain[T](r: (T, T) -> Bool, s: (T, T) -> Bool) {
    relation_subset(r, s) and has_no_descending_chain(s) implies has_no_descending_chain(r)
} by {
    if relation_subset(r, s) and has_no_descending_chain(s) {
        if has_descending_chain(r) {
            relation_subset_has_descending_chain(r, s)
            has_descending_chain(s)
            not has_descending_chain(s)
            false
        }
        not has_descending_chain(r)
        has_no_descending_chain(r)
    }
}

/// The intersection of a relation with a relation that has no descending chain also has no descending chain.
theorem relation_intersection_left_has_no_descending_chain[T](r: (T, T) -> Bool, s: (T, T) -> Bool) {
    has_no_descending_chain(r) implies has_no_descending_chain(relation_intersection(r, s))
} by {
    if has_no_descending_chain(r) {
        relation_intersection_left_subset(r, s)
        relation_subset_has_no_descending_chain(relation_intersection(r, s), r)
        has_no_descending_chain(relation_intersection(r, s))
    }
}

/// The intersection of any relation with a relation that has no descending chain also has no descending chain.
theorem relation_intersection_right_has_no_descending_chain[T](r: (T, T) -> Bool, s: (T, T) -> Bool) {
    has_no_descending_chain(s) implies has_no_descending_chain(relation_intersection(r, s))
} by {
    if has_no_descending_chain(s) {
        relation_intersection_right_subset(r, s)
        relation_subset_has_no_descending_chain(relation_intersection(r, s), s)
        has_no_descending_chain(relation_intersection(r, s))
    }
}

/// Absence of descending chains is invariant under mutual relation inclusion.
theorem relation_subset_both_has_no_descending_chain[T](r: (T, T) -> Bool, s: (T, T) -> Bool) {
    relation_subset(r, s) and relation_subset(s, r) and has_no_descending_chain(s) implies
    has_no_descending_chain(r)
} by {
    if relation_subset(r, s) and relation_subset(s, r) and has_no_descending_chain(s) {
        relation_subset_has_no_descending_chain(r, s)
        has_no_descending_chain(r)
    }
}

/// Pullback chains map to descending chains in the target relation.
theorem relation_pullback_descending_chain_map_is_descending[A, B](
    f: A -> B,
    r: (B, B) -> Bool,
    c: Nat -> A
) {
    is_descending_chain(relation_pullback(f, r), c) implies
    is_descending_chain(r, descending_chain_map(f, c))
} by {
    if is_descending_chain(relation_pullback(f, r), c) {
        is_descending_chain(relation_pullback(f, r), c) = forall(n: Nat) {
            relation_pullback(f, r, c(n.suc), c(n))
        }
        forall(n: Nat) {
            relation_pullback(f, r, c(n.suc), c(n))
            r(f(c(n.suc)), f(c(n)))
            descending_chain_map(f, c, n.suc) = f(c(n.suc))
            descending_chain_map(f, c, n) = f(c(n))
            r(descending_chain_map(f, c, n.suc), descending_chain_map(f, c, n))
        }
        is_descending_chain(r, descending_chain_map(f, c))
    }
}

/// A first-coordinate product descending chain projects to a descending chain.
theorem product_first_relation_descending_chain_first[A, B](
    r: (A, A) -> Bool,
    c: Nat -> Pair[A, B]
) {
    is_descending_chain(product_first_relation[A, B](r), c) implies
    is_descending_chain(r, descending_chain_first(c))
} by {
    if is_descending_chain(product_first_relation[A, B](r), c) {
        is_descending_chain(product_first_relation[A, B](r), c) = forall(n: Nat) {
            product_first_relation[A, B](r, c(n.suc), c(n))
        }
        forall(n: Nat) {
            product_first_relation[A, B](r, c(n.suc), c(n))
            r(c(n.suc).first, c(n).first)
            descending_chain_first(c, n.suc) = c(n.suc).first
            descending_chain_first(c, n) = c(n).first
            r(descending_chain_first(c, n.suc), descending_chain_first(c, n))
        }
        is_descending_chain(r, descending_chain_first(c))
    }
}

/// A second-coordinate product descending chain projects to a descending chain.
theorem product_second_relation_descending_chain_second[A, B](
    s: (B, B) -> Bool,
    c: Nat -> Pair[A, B]
) {
    is_descending_chain(product_second_relation[A, B](s), c) implies
    is_descending_chain(s, descending_chain_second(c))
} by {
    if is_descending_chain(product_second_relation[A, B](s), c) {
        is_descending_chain(product_second_relation[A, B](s), c) = forall(n: Nat) {
            product_second_relation[A, B](s, c(n.suc), c(n))
        }
        forall(n: Nat) {
            product_second_relation[A, B](s, c(n.suc), c(n))
            s(c(n.suc).second, c(n).second)
            descending_chain_second(c, n.suc) = c(n.suc).second
            descending_chain_second(c, n) = c(n).second
            s(descending_chain_second(c, n.suc), descending_chain_second(c, n))
        }
        is_descending_chain(s, descending_chain_second(c))
    }
}

/// A first-coordinate product descending chain gives a descending chain in the first relation.
theorem product_first_relation_has_descending_chain_imp[A, B](r: (A, A) -> Bool) {
    has_descending_chain(product_first_relation[A, B](r)) implies has_descending_chain(r)
} by {
    if has_descending_chain(product_first_relation[A, B](r)) {
        let c: Nat -> Pair[A, B] satisfy {
            is_descending_chain(product_first_relation[A, B](r), c)
        }
        product_first_relation_descending_chain_first(r, c)
        is_descending_chain(r, descending_chain_first(c))
        exists(d: Nat -> A) {
            is_descending_chain(r, d)
        }
        has_descending_chain(r)
    }
}

/// A second-coordinate product descending chain gives a descending chain in the second relation.
theorem product_second_relation_has_descending_chain_imp[A, B](s: (B, B) -> Bool) {
    has_descending_chain(product_second_relation[A, B](s)) implies has_descending_chain(s)
} by {
    if has_descending_chain(product_second_relation[A, B](s)) {
        let c: Nat -> Pair[A, B] satisfy {
            is_descending_chain(product_second_relation[A, B](s), c)
        }
        product_second_relation_descending_chain_second(s, c)
        is_descending_chain(s, descending_chain_second(c))
        exists(d: Nat -> B) {
            is_descending_chain(s, d)
        }
        has_descending_chain(s)
    }
}

/// Absence of first-coordinate descending chains blocks product descending chains.
theorem product_first_relation_has_no_descending_chain_of_component[A, B](r: (A, A) -> Bool) {
    has_no_descending_chain(r) implies has_no_descending_chain(product_first_relation[A, B](r))
} by {
    if has_no_descending_chain(r) {
        if has_descending_chain(product_first_relation[A, B](r)) {
            product_first_relation_has_descending_chain_imp[A, B](r)
            has_descending_chain(r)
            not has_descending_chain(r)
            false
        }
        not has_descending_chain(product_first_relation[A, B](r))
        has_no_descending_chain(product_first_relation[A, B](r))
    }
}

/// Absence of second-coordinate descending chains blocks product descending chains.
theorem product_second_relation_has_no_descending_chain_of_component[A, B](s: (B, B) -> Bool) {
    has_no_descending_chain(s) implies has_no_descending_chain(product_second_relation[A, B](s))
} by {
    if has_no_descending_chain(s) {
        if has_descending_chain(product_second_relation[A, B](s)) {
            product_second_relation_has_descending_chain_imp[A, B](s)
            has_descending_chain(s)
            not has_descending_chain(s)
            false
        }
        not has_descending_chain(product_second_relation[A, B](s))
        has_no_descending_chain(product_second_relation[A, B](s))
    }
}

/// A pullback relation has no descending chain when the target relation has none.
theorem relation_pullback_has_no_descending_chain[A, B](f: A -> B, r: (B, B) -> Bool) {
    has_no_descending_chain(r) implies has_no_descending_chain(relation_pullback(f, r))
} by {
    if has_no_descending_chain(r) {
        if has_descending_chain(relation_pullback(f, r)) {
            let c: Nat -> A satisfy {
                is_descending_chain(relation_pullback(f, r), c)
            }
            relation_pullback_descending_chain_map_is_descending(f, r, c)
            is_descending_chain(r, descending_chain_map(f, c))
            has_descending_chain(r)
            not has_descending_chain(r)
            false
        }
        not has_descending_chain(relation_pullback(f, r))
        has_no_descending_chain(relation_pullback(f, r))
    }
}

/// A pullback relation has a descending chain only if the target relation has one.
theorem relation_pullback_has_descending_chain_imp[A, B](f: A -> B, r: (B, B) -> Bool) {
    has_descending_chain(relation_pullback(f, r)) implies has_descending_chain(r)
} by {
    if has_descending_chain(relation_pullback(f, r)) {
        let c: Nat -> A satisfy {
            is_descending_chain(relation_pullback(f, r), c)
        }
        relation_pullback_descending_chain_map_is_descending(f, r, c)
        is_descending_chain(r, descending_chain_map(f, c))
        exists(d: Nat -> B) {
            is_descending_chain(r, d)
        }
        has_descending_chain(r)
    }
}

/// A chain for a relation controlled by a pullback maps to a descending chain in the target.
theorem relation_subset_pullback_descending_chain_map_is_descending[A, B](
    f: A -> B,
    r: (A, A) -> Bool,
    s: (B, B) -> Bool,
    c: Nat -> A
) {
    relation_subset(r, relation_pullback(f, s)) and is_descending_chain(r, c) implies
    is_descending_chain(s, descending_chain_map(f, c))
} by {
    if relation_subset(r, relation_pullback(f, s)) and is_descending_chain(r, c) {
        relation_subset_descending_chain(r, relation_pullback(f, s), c)
        is_descending_chain(relation_pullback(f, s), c)
        relation_pullback_descending_chain_map_is_descending(f, s, c)
        is_descending_chain(s, descending_chain_map(f, c))
    }
}

/// A relation controlled by a pullback has no descending chain when the target relation has none.
theorem relation_subset_pullback_has_no_descending_chain[A, B](
    f: A -> B,
    r: (A, A) -> Bool,
    s: (B, B) -> Bool
) {
    relation_subset(r, relation_pullback(f, s)) and has_no_descending_chain(s) implies
    has_no_descending_chain(r)
} by {
    if relation_subset(r, relation_pullback(f, s)) and has_no_descending_chain(s) {
        relation_pullback_has_no_descending_chain(f, s)
        has_no_descending_chain(relation_pullback(f, s))
        relation_subset_has_no_descending_chain(r, relation_pullback(f, s))
        has_no_descending_chain(r)
    }
}

/// A relation controlled by a pullback has a descending chain only if the target relation has one.
theorem relation_subset_pullback_has_descending_chain_imp[A, B](
    f: A -> B,
    r: (A, A) -> Bool,
    s: (B, B) -> Bool
) {
    relation_subset(r, relation_pullback(f, s)) and has_descending_chain(r) implies has_descending_chain(s)
} by {
    if relation_subset(r, relation_pullback(f, s)) and has_descending_chain(r) {
        relation_subset_has_descending_chain(r, relation_pullback(f, s))
        has_descending_chain(relation_pullback(f, s))
        relation_pullback_has_descending_chain_imp(f, s)
        has_descending_chain(s)
    }
}

/// A relation controlled by a well-founded target relation has no descending chain.
theorem relation_subset_pullback_well_founded_has_no_descending_chain[A, B](
    f: A -> B,
    r: (A, A) -> Bool,
    s: (B, B) -> Bool
) {
    relation_subset(r, relation_pullback(f, s)) and is_well_founded(s) implies has_no_descending_chain(r)
} by {
    if relation_subset(r, relation_pullback(f, s)) and is_well_founded(s) {
        well_founded_has_no_descending_chain_exists(s)
        has_no_descending_chain(s)
        relation_subset_pullback_has_no_descending_chain(f, r, s)
        has_no_descending_chain(r)
    }
}

/// A pushforward has no descending chain only when the source relation has none.
theorem relation_pushforward_has_no_descending_chain_reflects_source[A, B](f: A -> B, r: (A, A) -> Bool) {
    has_no_descending_chain(relation_pushforward(f, r)) implies has_no_descending_chain(r)
} by {
    if has_no_descending_chain(relation_pushforward(f, r)) {
        relation_subset_pullback_pushforward(f, r)
        relation_pullback_has_no_descending_chain(f, relation_pushforward(f, r))
        has_no_descending_chain(relation_pullback(f, relation_pushforward(f, r)))
        relation_subset_has_no_descending_chain(r, relation_pullback(f, relation_pushforward(f, r)))
        has_no_descending_chain(r)
    }
}

/// The first-coordinate product relation is well-founded when the first relation is.
theorem product_first_relation_is_well_founded[A, B](r: (A, A) -> Bool) {
    is_well_founded(r) implies is_well_founded(product_first_relation[A, B](r))
} by {
    if is_well_founded(r) {
        let first_projection: Pair[A, B] -> A = function(p: Pair[A, B]) {
            p.first
        }
        product_first_relation[A, B](r) = relation_pullback(first_projection, r)
        relation_pullback_is_well_founded(first_projection, r)
        is_well_founded(product_first_relation[A, B](r))
    }
}

/// The first-coordinate product relation has no descending chain when the first relation has none.
theorem product_first_relation_has_no_descending_chain[A, B](r: (A, A) -> Bool) {
    has_no_descending_chain(r) implies has_no_descending_chain(product_first_relation[A, B](r))
} by {
    if has_no_descending_chain(r) {
        let first_projection: Pair[A, B] -> A = function(p: Pair[A, B]) {
            p.first
        }
        product_first_relation[A, B](r) = relation_pullback(first_projection, r)
        relation_pullback_has_no_descending_chain(first_projection, r)
        has_no_descending_chain(product_first_relation[A, B](r))
    }
}

/// The first-coordinate product relation has no descending chain when the first relation is well-founded.
theorem product_first_relation_well_founded_has_no_descending_chain[A, B](r: (A, A) -> Bool) {
    is_well_founded(r) implies has_no_descending_chain(product_first_relation[A, B](r))
} by {
    if is_well_founded(r) {
        well_founded_has_no_descending_chain_exists(r)
        has_no_descending_chain(r)
        product_first_relation_has_no_descending_chain[A, B](r)
        has_no_descending_chain(product_first_relation[A, B](r))
    }
}

/// The second-coordinate product relation is well-founded when the second relation is.
theorem product_second_relation_is_well_founded[A, B](s: (B, B) -> Bool) {
    is_well_founded(s) implies is_well_founded(product_second_relation[A, B](s))
} by {
    if is_well_founded(s) {
        let second_projection: Pair[A, B] -> B = function(p: Pair[A, B]) {
            p.second
        }
        product_second_relation[A, B](s) = relation_pullback(second_projection, s)
        relation_pullback_is_well_founded(second_projection, s)
        is_well_founded(product_second_relation[A, B](s))
    }
}

/// The second-coordinate product relation has no descending chain when the second relation has none.
theorem product_second_relation_has_no_descending_chain[A, B](s: (B, B) -> Bool) {
    has_no_descending_chain(s) implies has_no_descending_chain(product_second_relation[A, B](s))
} by {
    if has_no_descending_chain(s) {
        let second_projection: Pair[A, B] -> B = function(p: Pair[A, B]) {
            p.second
        }
        product_second_relation[A, B](s) = relation_pullback(second_projection, s)
        relation_pullback_has_no_descending_chain(second_projection, s)
        has_no_descending_chain(product_second_relation[A, B](s))
    }
}

/// The second-coordinate product relation has no descending chain when the second relation is well-founded.
theorem product_second_relation_well_founded_has_no_descending_chain[A, B](s: (B, B) -> Bool) {
    is_well_founded(s) implies has_no_descending_chain(product_second_relation[A, B](s))
} by {
    if is_well_founded(s) {
        well_founded_has_no_descending_chain_exists(s)
        has_no_descending_chain(s)
        product_second_relation_has_no_descending_chain[A, B](s)
        has_no_descending_chain(product_second_relation[A, B](s))
    }
}

/// The first-coordinate product relation is Noetherian when the first relation is.
theorem product_first_relation_is_noetherian_relation[A, B](r: (A, A) -> Bool) {
    is_noetherian_relation(r) implies is_noetherian_relation(product_first_relation[A, B](r))
} by {
    if is_noetherian_relation(r) {
        let first_projection: Pair[A, B] -> A = function(p: Pair[A, B]) {
            p.first
        }
        product_first_relation[A, B](r) = relation_pullback(first_projection, r)
        relation_pullback_is_noetherian_relation(first_projection, r)
        is_noetherian_relation(product_first_relation[A, B](r))
    }
}

/// The first-coordinate product relation has no ascending chain when the first relation has none.
theorem product_first_relation_has_no_ascending_chain[A, B](r: (A, A) -> Bool) {
    has_no_ascending_chain(r) implies has_no_ascending_chain(product_first_relation[A, B](r))
} by {
    if has_no_ascending_chain(r) {
        if has_ascending_chain(product_first_relation[A, B](r)) {
            let c: Nat -> Pair[A, B] satisfy {
                is_ascending_chain(product_first_relation[A, B](r), c)
            }
            is_ascending_chain(product_first_relation[A, B](r), c) = forall(k: Nat) {
                product_first_relation[A, B](r, c(k), c(k.suc))
            }
            forall(n: Nat) {
                product_first_relation[A, B](r, c(n), c(n.suc))
                r(c(n).first, c(n.suc).first)
                descending_chain_first(c, n.suc) = c(n.suc).first
                descending_chain_first(c, n) = c(n).first
                r(descending_chain_first(c, n), descending_chain_first(c, n.suc))
            }
            is_ascending_chain(r, descending_chain_first(c))
            has_ascending_chain(r)
            not has_ascending_chain(r)
            false
        }
        not has_ascending_chain(product_first_relation[A, B](r))
        has_no_ascending_chain(product_first_relation[A, B](r))
    }
}

/// The first-coordinate product relation has no ascending chain when the first relation is Noetherian.
theorem product_first_relation_noetherian_has_no_ascending_chain[A, B](r: (A, A) -> Bool) {
    is_noetherian_relation(r) implies has_no_ascending_chain(product_first_relation[A, B](r))
} by {
    if is_noetherian_relation(r) {
        noetherian_relation_has_no_ascending_chain(r)
        has_no_ascending_chain(r)
        product_first_relation_has_no_ascending_chain[A, B](r)
        has_no_ascending_chain(product_first_relation[A, B](r))
    }
}

/// The second-coordinate product relation is Noetherian when the second relation is.
theorem product_second_relation_is_noetherian_relation[A, B](s: (B, B) -> Bool) {
    is_noetherian_relation(s) implies is_noetherian_relation(product_second_relation[A, B](s))
} by {
    if is_noetherian_relation(s) {
        let second_projection: Pair[A, B] -> B = function(p: Pair[A, B]) {
            p.second
        }
        product_second_relation[A, B](s) = relation_pullback(second_projection, s)
        relation_pullback_is_noetherian_relation(second_projection, s)
        is_noetherian_relation(product_second_relation[A, B](s))
    }
}

/// The second-coordinate product relation has no ascending chain when the second relation has none.
theorem product_second_relation_has_no_ascending_chain[A, B](s: (B, B) -> Bool) {
    has_no_ascending_chain(s) implies has_no_ascending_chain(product_second_relation[A, B](s))
} by {
    if has_no_ascending_chain(s) {
        if has_ascending_chain(product_second_relation[A, B](s)) {
            let c: Nat -> Pair[A, B] satisfy {
                is_ascending_chain(product_second_relation[A, B](s), c)
            }
            is_ascending_chain(product_second_relation[A, B](s), c) = forall(k: Nat) {
                product_second_relation[A, B](s, c(k), c(k.suc))
            }
            forall(n: Nat) {
                product_second_relation[A, B](s, c(n), c(n.suc))
                s(c(n).second, c(n.suc).second)
                descending_chain_second(c, n.suc) = c(n.suc).second
                descending_chain_second(c, n) = c(n).second
                s(descending_chain_second(c, n), descending_chain_second(c, n.suc))
            }
            is_ascending_chain(s, descending_chain_second(c))
            has_ascending_chain(s)
            not has_ascending_chain(s)
            false
        }
        not has_ascending_chain(product_second_relation[A, B](s))
        has_no_ascending_chain(product_second_relation[A, B](s))
    }
}

/// The second-coordinate product relation has no ascending chain when the second relation is Noetherian.
theorem product_second_relation_noetherian_has_no_ascending_chain[A, B](s: (B, B) -> Bool) {
    is_noetherian_relation(s) implies has_no_ascending_chain(product_second_relation[A, B](s))
} by {
    if is_noetherian_relation(s) {
        noetherian_relation_has_no_ascending_chain(s)
        has_no_ascending_chain(s)
        product_second_relation_has_no_ascending_chain[A, B](s)
        has_no_ascending_chain(product_second_relation[A, B](s))
    }
}

/// The lexicographic relation is well-founded when both component relations are well-founded.
theorem lex_relation_is_well_founded[A, B](r: (A, A) -> Bool, s: (B, B) -> Bool) {
    is_well_founded(r) and is_well_founded(s) implies is_well_founded(lex_relation(r, s))
} by {
    if is_well_founded(r) and is_well_founded(s) {
        is_well_founded(r) = forall(p: A -> Bool) {
            (exists(x: A) { p(x) }) implies exists(m: A) {
                p(m) and forall(y: A) {
                    r(y, m) implies not p(y)
                }
            }
        }
        is_well_founded(s) = forall(p: B -> Bool) {
            (exists(x: B) { p(x) }) implies exists(m: B) {
                p(m) and forall(y: B) {
                    s(y, m) implies not p(y)
                }
            }
        }
        forall(p: Pair[A, B] -> Bool) {
            if exists(x: Pair[A, B]) { p(x) } {
                let first_pred: A -> Bool = function(a: A) {
                    exists(b: B) {
                        p(Pair.new(a, b))
                    }
                }
                let point: Pair[A, B] satisfy {
                    p(point)
                }
                pair_eta(point)
                p(Pair.new(point.first, point.second))
                first_pred(point.first)
                exists(a: A) {
                    first_pred(a)
                }
                exists(a0: A) {
                    first_pred(a0) and forall(a: A) {
                        r(a, a0) implies not first_pred(a)
                    }
                }
                let a0: A satisfy {
                    first_pred(a0) and forall(a: A) {
                        r(a, a0) implies not first_pred(a)
                    }
                }
                let second_pred: B -> Bool = function(b: B) {
                    p(Pair.new(a0, b))
                }
                let b_witness: B satisfy {
                    p(Pair.new(a0, b_witness))
                }
                second_pred(b_witness)
                exists(b: B) {
                    second_pred(b)
                }
                exists(b0: B) {
                    second_pred(b0) and forall(b: B) {
                        s(b, b0) implies not second_pred(b)
                    }
                }
                let b0: B satisfy {
                    second_pred(b0) and forall(b: B) {
                        s(b, b0) implies not second_pred(b)
                    }
                }
                let m: Pair[A, B] = Pair.new(a0, b0)
                forall(y: Pair[A, B]) {
                    if lex_relation(r, s, y, m) {
                        lex_relation(r, s, y, m) =
                            (r(y.first, m.first) or
                             (y.first = m.first and s(y.second, m.second)))
                        r(y.first, m.first) or
                            (y.first = m.first and s(y.second, m.second))
                        if p(y) {
                            if r(y.first, m.first) {
                                m.first = a0
                                r(y.first, a0)
                                forall(a: A) {
                                    r(a, a0) implies not first_pred(a)
                                }
                                not first_pred(y.first)
                                pair_eta(y)
                                p(Pair.new(y.first, y.second))
                                first_pred(y.first)
                                false
                            }
                            if y.first = m.first and s(y.second, m.second) {
                                m.first = a0
                                m.second = b0
                                y.first = a0
                                s(y.second, b0)
                                forall(b: B) {
                                    s(b, b0) implies not second_pred(b)
                                }
                                not second_pred(y.second)
                                pair_eta(y)
                                p(Pair.new(y.first, y.second))
                                p(Pair.new(a0, y.second))
                                second_pred(y.second)
                                false
                            }
                            false
                        }
                        not p(y)
                    }
                }
                p(m)
                p(m) and forall(y: Pair[A, B]) {
                    lex_relation(r, s, y, m) implies not p(y)
                }
                exists(minimal: Pair[A, B]) {
                    p(minimal) and forall(y: Pair[A, B]) {
                        lex_relation(r, s, y, minimal) implies not p(y)
                    }
                }
            }
        }
        is_well_founded(lex_relation(r, s)) = forall(p: Pair[A, B] -> Bool) {
            (exists(x: Pair[A, B]) { p(x) }) implies exists(m: Pair[A, B]) {
                p(m) and forall(y: Pair[A, B]) {
                    lex_relation(r, s, y, m) implies not p(y)
                }
            }
        }
        forall(p: Pair[A, B] -> Bool) {
            (exists(x: Pair[A, B]) { p(x) }) implies exists(m: Pair[A, B]) {
                p(m) and forall(y: Pair[A, B]) {
                    lex_relation(r, s, y, m) implies not p(y)
                }
            }
        }
    }
}

/// A lexicographic relation has no descending chain when both component relations are well-founded.
theorem lex_relation_well_founded_has_no_descending_chain[A, B](r: (A, A) -> Bool, s: (B, B) -> Bool) {
    is_well_founded(r) and is_well_founded(s) implies has_no_descending_chain(lex_relation(r, s))
} by {
    if is_well_founded(r) and is_well_founded(s) {
        lex_relation_is_well_founded(r, s)
        is_well_founded(lex_relation(r, s))
        well_founded_has_no_descending_chain_exists(lex_relation(r, s))
        has_no_descending_chain(lex_relation(r, s))
    }
}

/// The converse of a lexicographic relation is the lexicographic relation of the converses.
theorem relation_converse_lex_relation[A, B](r: (A, A) -> Bool, s: (B, B) -> Bool) {
    relation_converse(lex_relation(r, s)) =
    lex_relation(relation_converse(r), relation_converse(s))
} by {
    let u = relation_converse(lex_relation(r, s))
    let v = lex_relation(relation_converse(r), relation_converse(s))
    forall(p: Pair[A, B], q: Pair[A, B]) {
        u(p, q) = lex_relation(r, s, q, p)
        v(p, q) =
            (relation_converse(r, p.first, q.first) or
             (p.first = q.first and relation_converse(s, p.second, q.second)))
        relation_converse(r, p.first, q.first) = r(q.first, p.first)
        relation_converse(s, p.second, q.second) = s(q.second, p.second)
        lex_relation(r, s, q, p) =
            (r(q.first, p.first) or (q.first = p.first and s(q.second, p.second)))
        if u(p, q) {
            if r(q.first, p.first) {
                relation_converse(r, p.first, q.first)
                v(p, q)
            }
            if q.first = p.first and s(q.second, p.second) {
                p.first = q.first
                relation_converse(s, p.second, q.second)
                p.first = q.first and relation_converse(s, p.second, q.second)
                v(p, q)
            }
            v(p, q)
        }
        if v(p, q) {
            if relation_converse(r, p.first, q.first) {
                r(q.first, p.first)
                u(p, q)
            }
            if p.first = q.first and relation_converse(s, p.second, q.second) {
                q.first = p.first
                s(q.second, p.second)
                q.first = p.first and s(q.second, p.second)
                u(p, q)
            }
            u(p, q)
        }
        u(p, q) = v(p, q)
    }
    binary_function_extensionality(u, v)
}

/// A lexicographic relation is Noetherian when both component relations are Noetherian.
theorem lex_relation_is_noetherian_relation[A, B](r: (A, A) -> Bool, s: (B, B) -> Bool) {
    is_noetherian_relation(r) and is_noetherian_relation(s) implies
    is_noetherian_relation(lex_relation(r, s))
} by {
    if is_noetherian_relation(r) and is_noetherian_relation(s) {
        is_noetherian_relation(r) = is_well_founded(relation_converse(r))
        is_noetherian_relation(s) = is_well_founded(relation_converse(s))
        lex_relation_is_well_founded(relation_converse(r), relation_converse(s))
        is_well_founded(lex_relation(relation_converse(r), relation_converse(s)))
        relation_converse_lex_relation(r, s)
        is_well_founded(relation_converse(lex_relation(r, s)))
        is_noetherian_relation(lex_relation(r, s))
    }
}

/// A lexicographic relation has no ascending chain when both component relations are Noetherian.
theorem lex_relation_noetherian_has_no_ascending_chain[A, B](r: (A, A) -> Bool, s: (B, B) -> Bool) {
    is_noetherian_relation(r) and is_noetherian_relation(s) implies
    has_no_ascending_chain(lex_relation(r, s))
} by {
    if is_noetherian_relation(r) and is_noetherian_relation(s) {
        lex_relation_is_noetherian_relation(r, s)
        is_noetherian_relation(lex_relation(r, s))
        noetherian_relation_has_no_ascending_chain(lex_relation(r, s))
        has_no_ascending_chain(lex_relation(r, s))
    }
}

/// Each lexicographic-descending chain projects to a non-strictly descending chain
/// of first coordinates: each successor first coordinate is either equal or `r`-below
/// the previous.
theorem lex_relation_descending_chain_first[A, B](
    r: (A, A) -> Bool,
    s: (B, B) -> Bool,
    c: Nat -> Pair[A, B],
    n: Nat
) {
    is_descending_chain(lex_relation(r, s), c) implies
        c(n.suc).first = c(n).first or r(c(n.suc).first, c(n).first)
} by {
    if is_descending_chain(lex_relation(r, s), c) {
        is_descending_chain(lex_relation(r, s), c) = forall(k: Nat) {
            lex_relation(r, s, c(k.suc), c(k))
        }
        lex_relation(r, s, c(n.suc), c(n))
        lex_relation(r, s, c(n.suc), c(n)) =
            (r(c(n.suc).first, c(n).first) or
             (c(n.suc).first = c(n).first and s(c(n.suc).second, c(n).second)))
        r(c(n.suc).first, c(n).first) or
            (c(n.suc).first = c(n).first and s(c(n.suc).second, c(n).second))
        if r(c(n.suc).first, c(n).first) {
            c(n.suc).first = c(n).first or r(c(n.suc).first, c(n).first)
        }
        if c(n.suc).first = c(n).first and s(c(n.suc).second, c(n).second) {
            c(n.suc).first = c(n).first
            c(n.suc).first = c(n).first or r(c(n.suc).first, c(n).first)
        }
        c(n.suc).first = c(n).first or r(c(n.suc).first, c(n).first)
    }
}

/// Each lexicographic-descending chain that holds the first coordinate constant on
/// a successor step has its second coordinates `s`-descending on that step.
theorem lex_relation_descending_chain_second_when_same_first[A, B](
    r: (A, A) -> Bool,
    s: (B, B) -> Bool,
    c: Nat -> Pair[A, B],
    n: Nat
) {
    is_descending_chain(lex_relation(r, s), c) and
    c(n.suc).first = c(n).first and
    is_irreflexive(r) implies
        s(c(n.suc).second, c(n).second)
} by {
    if is_descending_chain(lex_relation(r, s), c) and
        c(n.suc).first = c(n).first and
        is_irreflexive(r) {
        is_descending_chain(lex_relation(r, s), c) = forall(k: Nat) {
            lex_relation(r, s, c(k.suc), c(k))
        }
        lex_relation(r, s, c(n.suc), c(n))
        lex_relation(r, s, c(n.suc), c(n)) =
            (r(c(n.suc).first, c(n).first) or
             (c(n.suc).first = c(n).first and s(c(n.suc).second, c(n).second)))
        if r(c(n.suc).first, c(n).first) {
            r(c(n).first, c(n).first)
            is_irreflexive(r) = forall(x: A) { not r(x, x) }
            not r(c(n).first, c(n).first)
            false
        }
        c(n.suc).first = c(n).first and s(c(n.suc).second, c(n).second)
        s(c(n.suc).second, c(n).second)
    }
}
