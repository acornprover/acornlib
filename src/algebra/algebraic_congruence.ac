/// Named congruence predicates for additional algebraic structures and
/// the corresponding kernel-of-homomorphism instances.

from algebra.add_monoid import AddMonoid, AddMonoidHom
from algebra.monoid.monoid import Monoid, MonoidHom
from algebra.add_group import AddGroup, AddGroupHom
from algebra.group import Group, GroupHom
from algebra.add_comm_group import AddCommGroup
from semiring import Semiring
from algebra.ring.ring import Ring
from algebra.module.module import Module
from algebra.module.module_hom import is_linear_map
from data.basic.relation_transport import is_add_congruence, is_mul_congruence,
    is_unary_congruence, is_binary_congruence, respects_unary_op, respects_binary_op, respects_add, respects_mul,
    add_congruence_is_equivalence, mul_congruence_is_equivalence,
    add_congruence_respects_add, mul_congruence_respects_mul,
    eq_relation_is_add_congruence, eq_relation_is_mul_congruence,
    eq_relation_is_unary_congruence, eq_relation_respects_unary_op
from data.basic.relation_basic import is_equivalence, eq_relation
from data.basic.equivalence import kernel_relation
from data.basic.quotient_algebra import add_monoid_hom_kernel_is_add_congruence,
    monoid_hom_kernel_is_mul_congruence,
    is_add_group_congruence, is_group_congruence, is_ring_congruence,
    is_semiring_congruence, linear_map_kernel_is_add_group_congruence,
    linear_map_kernel_respects_smul,
    add_group_hom_kernel_is_add_group_congruence,
    group_hom_kernel_is_group_congruence,
    semiring_hom_kernel_is_semiring_congruence,
    ring_hom_kernel_is_ring_congruence
from algebra.semiring_hom import SemiringHom
from algebra.ring.ring_hom import RingHom

/// True if a relation is an equivalence relation compatible with monoid
/// multiplication. The identity element is fixed, so no extra condition is
/// needed beyond multiplicative compatibility.
define is_monoid_congruence[M: Monoid](r: (M, M) -> Bool) -> Bool {
    is_mul_congruence(r)
}

/// True if a relation is an equivalence relation compatible with additive
/// monoid addition. The zero element is fixed, so no extra condition is
/// needed beyond additive compatibility.
define is_add_monoid_congruence[A: AddMonoid](r: (A, A) -> Bool) -> Bool {
    is_add_congruence(r)
}

/// The kernel of a monoid homomorphism is a monoid congruence.
theorem monoid_hom_kernel_is_monoid_congruence[M: Monoid, N: Monoid](f: MonoidHom[M, N]) {
    is_monoid_congruence(kernel_relation(f.hom))
} by {
    monoid_hom_kernel_is_mul_congruence(f)
}

/// The kernel of an additive monoid homomorphism is an additive-monoid congruence.
theorem add_monoid_hom_kernel_is_add_monoid_congruence[A: AddMonoid, B: AddMonoid](f: AddMonoidHom[A, B]) {
    is_add_monoid_congruence(kernel_relation(f.hom))
} by {
    add_monoid_hom_kernel_is_add_congruence(f.hom)
}

/// True if a relation on a module carrier is an additive-group congruence and
/// is compatible with scalar multiplication by every scalar.
define is_module_congruence[R: Ring, M: AddCommGroup](
    src: Module[R, M], r: (M, M) -> Bool
) -> Bool {
    is_add_group_congruence(r) and forall(s: R) {
        respects_unary_op(r, src.smul(s))
    }
}

/// The kernel of a linear map respects scalar multiplication for every scalar.
theorem linear_map_kernel_respects_all_smul[R: Ring, M: AddCommGroup, N: AddCommGroup](
    src: Module[R, M], dst: Module[R, N], f: M -> N
) {
    is_linear_map(src, dst, f) implies forall(s: R) {
        respects_unary_op(kernel_relation(f), src.smul(s))
    }
} by {
    if is_linear_map(src, dst, f) {
        forall(s: R) {
            linear_map_kernel_respects_smul(src, dst, f, s)
            respects_unary_op(kernel_relation(f), src.smul(s))
        }
    }
}

/// The kernel of a linear map is a module congruence on the source module.
theorem linear_map_kernel_is_module_congruence[R: Ring, M: AddCommGroup, N: AddCommGroup](
    src: Module[R, M], dst: Module[R, N], f: M -> N
) {
    is_linear_map(src, dst, f) implies is_module_congruence(src, kernel_relation(f))
} by {
    if is_linear_map(src, dst, f) {
        linear_map_kernel_is_add_group_congruence(src, dst, f)
        linear_map_kernel_respects_all_smul(src, dst, f)
        is_add_group_congruence(kernel_relation(f)) and forall(s: R) {
            respects_unary_op(kernel_relation(f), src.smul(s))
        }
    }
}

/// A module congruence is in particular an additive-group congruence.
theorem is_module_congruence_is_add_group_congruence[R: Ring, M: AddCommGroup](
    src: Module[R, M], r: (M, M) -> Bool
) {
    is_module_congruence(src, r) implies is_add_group_congruence(r)
} by {
    if is_module_congruence(src, r) {
        is_module_congruence(src, r) = (
            is_add_group_congruence(r) and forall(s: R) {
                respects_unary_op(r, src.smul(s))
            }
        )
    }
}

/// A module congruence respects scalar multiplication by every scalar.
theorem is_module_congruence_respects_smul[R: Ring, M: AddCommGroup](
    src: Module[R, M], r: (M, M) -> Bool, s: R
) {
    is_module_congruence(src, r) implies respects_unary_op(r, src.smul(s))
} by {
    if is_module_congruence(src, r) {
        is_module_congruence(src, r) = (
            is_add_group_congruence(r) and forall(t: R) {
                respects_unary_op(r, src.smul(t))
            }
        )
        forall(t: R) {
            respects_unary_op(r, src.smul(t))
        }
    }
}

/// Conjunctive characterization of a module congruence.
theorem is_module_congruence_iff[R: Ring, M: AddCommGroup](
    src: Module[R, M], r: (M, M) -> Bool
) {
    is_module_congruence(src, r) = (
        is_add_group_congruence(r) and forall(s: R) {
            respects_unary_op(r, src.smul(s))
        }
    )
}

/// A monoid congruence is in particular a multiplicative congruence.
theorem is_monoid_congruence_is_mul_congruence[M: Monoid](r: (M, M) -> Bool) {
    is_monoid_congruence(r) implies is_mul_congruence(r)
}

/// Monoid and multiplicative congruences coincide.
theorem is_monoid_congruence_eq_is_mul_congruence[M: Monoid](r: (M, M) -> Bool) {
    is_monoid_congruence(r) = is_mul_congruence(r)
}

/// Additive-monoid and additive congruences coincide.
theorem is_add_monoid_congruence_eq_is_add_congruence[A: AddMonoid](r: (A, A) -> Bool) {
    is_add_monoid_congruence(r) = is_add_congruence(r)
}

/// An additive-monoid congruence is in particular an additive congruence.
theorem is_add_monoid_congruence_is_add_congruence[A: AddMonoid](r: (A, A) -> Bool) {
    is_add_monoid_congruence(r) implies is_add_congruence(r)
}

/// A multiplicative congruence is in particular a monoid congruence.
theorem is_mul_congruence_is_monoid_congruence[M: Monoid](r: (M, M) -> Bool) {
    is_mul_congruence(r) implies is_monoid_congruence(r)
}

/// An additive congruence is in particular an additive-monoid congruence.
theorem is_add_congruence_is_add_monoid_congruence[A: AddMonoid](r: (A, A) -> Bool) {
    is_add_congruence(r) implies is_add_monoid_congruence(r)
}

/// Conjunctive characterization of an additive-group congruence.
theorem is_add_group_congruence_iff[A: AddGroup](r: (A, A) -> Bool) {
    is_add_group_congruence(r) = (is_add_congruence(r) and is_unary_congruence(r, A.neg))
}

/// An additive-group congruence is in particular an additive congruence.
theorem is_add_group_congruence_is_add_congruence[A: AddGroup](r: (A, A) -> Bool) {
    is_add_group_congruence(r) implies is_add_congruence(r)
} by {
    if is_add_group_congruence(r) {
        is_add_group_congruence(r) = (is_add_congruence(r) and is_unary_congruence(r, A.neg))
    }
}

/// An additive-group congruence respects negation.
theorem is_add_group_congruence_respects_neg[A: AddGroup](r: (A, A) -> Bool) {
    is_add_group_congruence(r) implies is_unary_congruence(r, A.neg)
} by {
    if is_add_group_congruence(r) {
        is_add_group_congruence(r) = (is_add_congruence(r) and is_unary_congruence(r, A.neg))
    }
}

/// An additive-group congruence is in particular an additive-monoid congruence.
theorem is_add_group_congruence_is_add_monoid_congruence[A: AddGroup](r: (A, A) -> Bool) {
    is_add_group_congruence(r) implies is_add_monoid_congruence(r)
} by {
    is_add_group_congruence_is_add_congruence(r)
}

/// Conjunctive characterization of a group congruence.
theorem is_group_congruence_iff[G: Group](r: (G, G) -> Bool) {
    is_group_congruence(r) = (is_mul_congruence(r) and is_unary_congruence(r, G.inverse))
}

/// A group congruence is in particular a multiplicative congruence.
theorem is_group_congruence_is_mul_congruence[G: Group](r: (G, G) -> Bool) {
    is_group_congruence(r) implies is_mul_congruence(r)
} by {
    if is_group_congruence(r) {
        is_group_congruence(r) = (is_mul_congruence(r) and is_unary_congruence(r, G.inverse))
    }
}

/// A group congruence respects inversion.
theorem is_group_congruence_respects_inverse[G: Group](r: (G, G) -> Bool) {
    is_group_congruence(r) implies is_unary_congruence(r, G.inverse)
} by {
    if is_group_congruence(r) {
        is_group_congruence(r) = (is_mul_congruence(r) and is_unary_congruence(r, G.inverse))
    }
}

/// A group congruence is in particular a monoid congruence.
theorem is_group_congruence_is_monoid_congruence[G: Group](r: (G, G) -> Bool) {
    is_group_congruence(r) implies is_monoid_congruence(r)
} by {
    is_group_congruence_is_mul_congruence(r)
}

/// Conjunctive characterization of a semiring congruence.
theorem is_semiring_congruence_iff[S: Semiring](r: (S, S) -> Bool) {
    is_semiring_congruence(r) = (is_add_congruence(r) and is_mul_congruence(r))
}

/// A semiring congruence is in particular an additive congruence.
theorem is_semiring_congruence_is_add_congruence[S: Semiring](r: (S, S) -> Bool) {
    is_semiring_congruence(r) implies is_add_congruence(r)
} by {
    if is_semiring_congruence(r) {
        is_semiring_congruence(r) = (is_add_congruence(r) and is_mul_congruence(r))
    }
}

/// A semiring congruence is in particular a multiplicative congruence.
theorem is_semiring_congruence_is_mul_congruence[S: Semiring](r: (S, S) -> Bool) {
    is_semiring_congruence(r) implies is_mul_congruence(r)
} by {
    if is_semiring_congruence(r) {
        is_semiring_congruence(r) = (is_add_congruence(r) and is_mul_congruence(r))
    }
}

/// A semiring congruence is in particular an additive-monoid congruence.
theorem is_semiring_congruence_is_add_monoid_congruence[S: Semiring](r: (S, S) -> Bool) {
    is_semiring_congruence(r) implies is_add_monoid_congruence(r)
} by {
    is_semiring_congruence_is_add_congruence(r)
}

/// A semiring congruence is in particular a monoid congruence.
theorem is_semiring_congruence_is_monoid_congruence[S: Semiring](r: (S, S) -> Bool) {
    is_semiring_congruence(r) implies is_monoid_congruence(r)
} by {
    is_semiring_congruence_is_mul_congruence(r)
}

/// Conjunctive characterization of a ring congruence.
theorem is_ring_congruence_iff[R: Ring](r: (R, R) -> Bool) {
    is_ring_congruence(r) = (is_add_congruence(r) and is_mul_congruence(r))
}

/// A ring congruence is in particular an additive congruence.
theorem is_ring_congruence_is_add_congruence[R: Ring](r: (R, R) -> Bool) {
    is_ring_congruence(r) implies is_add_congruence(r)
} by {
    if is_ring_congruence(r) {
        is_ring_congruence(r) = (is_add_congruence(r) and is_mul_congruence(r))
    }
}

/// A ring congruence is in particular a multiplicative congruence.
theorem is_ring_congruence_is_mul_congruence[R: Ring](r: (R, R) -> Bool) {
    is_ring_congruence(r) implies is_mul_congruence(r)
} by {
    if is_ring_congruence(r) {
        is_ring_congruence(r) = (is_add_congruence(r) and is_mul_congruence(r))
    }
}

/// A ring congruence is in particular a semiring congruence.
theorem is_ring_congruence_is_semiring_congruence[R: Ring](r: (R, R) -> Bool) {
    is_ring_congruence(r) implies is_semiring_congruence(r)
} by {
    if is_ring_congruence(r) {
        is_ring_congruence_is_add_congruence(r)
        is_ring_congruence_is_mul_congruence(r)
        is_semiring_congruence_iff(r)
    }
}

/// A ring congruence is in particular an additive-monoid congruence.
theorem is_ring_congruence_is_add_monoid_congruence[R: Ring](r: (R, R) -> Bool) {
    is_ring_congruence(r) implies is_add_monoid_congruence(r)
} by {
    is_ring_congruence_is_add_congruence(r)
}

/// A ring congruence is in particular a monoid congruence.
theorem is_ring_congruence_is_monoid_congruence[R: Ring](r: (R, R) -> Bool) {
    is_ring_congruence(r) implies is_monoid_congruence(r)
} by {
    is_ring_congruence_is_mul_congruence(r)
}

/// An additive-group congruence is built from an additive congruence that also respects negation.
theorem is_add_group_congruence_intro[A: AddGroup](r: (A, A) -> Bool) {
    is_add_congruence(r) and is_unary_congruence(r, A.neg) implies is_add_group_congruence(r)
} by {
    if is_add_congruence(r) and is_unary_congruence(r, A.neg) {
        is_add_group_congruence_iff(r)
    }
}

/// A group congruence is built from a multiplicative congruence that also respects inversion.
theorem is_group_congruence_intro[G: Group](r: (G, G) -> Bool) {
    is_mul_congruence(r) and is_unary_congruence(r, G.inverse) implies is_group_congruence(r)
} by {
    if is_mul_congruence(r) and is_unary_congruence(r, G.inverse) {
        is_group_congruence_iff(r)
    }
}

/// A semiring congruence is built from compatible additive and multiplicative congruences.
theorem is_semiring_congruence_intro[S: Semiring](r: (S, S) -> Bool) {
    is_add_congruence(r) and is_mul_congruence(r) implies is_semiring_congruence(r)
} by {
    if is_add_congruence(r) and is_mul_congruence(r) {
        is_semiring_congruence_iff(r)
    }
}

/// A ring congruence is built from compatible additive and multiplicative congruences.
theorem is_ring_congruence_intro[R: Ring](r: (R, R) -> Bool) {
    is_add_congruence(r) and is_mul_congruence(r) implies is_ring_congruence(r)
} by {
    if is_add_congruence(r) and is_mul_congruence(r) {
        is_ring_congruence_iff(r)
    }
}

/// A module congruence is built from an additive-group congruence together with
/// scalar-multiplication respect for every scalar.
theorem is_module_congruence_intro[R: Ring, M: AddCommGroup](
    src: Module[R, M], r: (M, M) -> Bool
) {
    is_add_group_congruence(r) and forall(s: R) {
        respects_unary_op(r, src.smul(s))
    } implies is_module_congruence(src, r)
} by {
    if is_add_group_congruence(r) and forall(s: R) { respects_unary_op(r, src.smul(s)) } {
        is_module_congruence_iff(src, r)
    }
}

/// A semiring congruence on a ring carrier is also a ring congruence.
theorem is_semiring_congruence_is_ring_congruence[R: Ring](r: (R, R) -> Bool) {
    is_semiring_congruence(r) implies is_ring_congruence(r)
} by {
    if is_semiring_congruence(r) {
        is_semiring_congruence_is_add_congruence(r)
        is_semiring_congruence_is_mul_congruence(r)
        is_ring_congruence_iff(r)
    }
}

/// A monoid congruence is built from a multiplicative congruence (the definitions agree).
theorem is_monoid_congruence_intro[M: Monoid](r: (M, M) -> Bool) {
    is_mul_congruence(r) implies is_monoid_congruence(r)
}

/// An additive-monoid congruence is built from an additive congruence (the definitions agree).
theorem is_add_monoid_congruence_intro[A: AddMonoid](r: (A, A) -> Bool) {
    is_add_congruence(r) implies is_add_monoid_congruence(r)
}

/// A monoid congruence is an equivalence relation.
theorem is_monoid_congruence_is_equivalence[M: Monoid](r: (M, M) -> Bool) {
    is_monoid_congruence(r) implies is_equivalence(r)
} by {
    if is_monoid_congruence(r) {
        is_monoid_congruence_is_mul_congruence(r)
        mul_congruence_is_equivalence(r)
    }
}

/// An additive-monoid congruence is an equivalence relation.
theorem is_add_monoid_congruence_is_equivalence[A: AddMonoid](r: (A, A) -> Bool) {
    is_add_monoid_congruence(r) implies is_equivalence(r)
} by {
    if is_add_monoid_congruence(r) {
        is_add_monoid_congruence_is_add_congruence(r)
        add_congruence_is_equivalence(r)
    }
}

/// An additive-group congruence is an equivalence relation.
theorem is_add_group_congruence_is_equivalence[A: AddGroup](r: (A, A) -> Bool) {
    is_add_group_congruence(r) implies is_equivalence(r)
} by {
    if is_add_group_congruence(r) {
        is_add_group_congruence_is_add_congruence(r)
        add_congruence_is_equivalence(r)
    }
}

/// A group congruence is an equivalence relation.
theorem is_group_congruence_is_equivalence[G: Group](r: (G, G) -> Bool) {
    is_group_congruence(r) implies is_equivalence(r)
} by {
    if is_group_congruence(r) {
        is_group_congruence_is_mul_congruence(r)
        mul_congruence_is_equivalence(r)
    }
}

/// A semiring congruence is an equivalence relation.
theorem is_semiring_congruence_is_equivalence[S: Semiring](r: (S, S) -> Bool) {
    is_semiring_congruence(r) implies is_equivalence(r)
} by {
    if is_semiring_congruence(r) {
        is_semiring_congruence_is_add_congruence(r)
        add_congruence_is_equivalence(r)
    }
}

/// A ring congruence is an equivalence relation.
theorem is_ring_congruence_is_equivalence[R: Ring](r: (R, R) -> Bool) {
    is_ring_congruence(r) implies is_equivalence(r)
} by {
    if is_ring_congruence(r) {
        is_ring_congruence_is_add_congruence(r)
        add_congruence_is_equivalence(r)
    }
}

/// The equality relation on a monoid is a monoid congruence.
theorem eq_relation_is_monoid_congruence[M: Monoid] {
    is_monoid_congruence(eq_relation[M])
} by {
    eq_relation_is_mul_congruence[M]
}

/// The equality relation on an additive monoid is an additive-monoid congruence.
theorem eq_relation_is_add_monoid_congruence[A: AddMonoid] {
    is_add_monoid_congruence(eq_relation[A])
} by {
    eq_relation_is_add_congruence[A]
}

/// The equality relation on an additive group is an additive-group congruence.
theorem eq_relation_is_add_group_congruence[A: AddGroup] {
    is_add_group_congruence(eq_relation[A])
} by {
    eq_relation_is_add_congruence[A]
    eq_relation_is_unary_congruence(A.neg)
    is_add_group_congruence_intro(eq_relation[A])
}

/// The equality relation on a group is a group congruence.
theorem eq_relation_is_group_congruence[G: Group] {
    is_group_congruence(eq_relation[G])
} by {
    eq_relation_is_mul_congruence[G]
    eq_relation_is_unary_congruence(G.inverse)
    is_group_congruence_intro(eq_relation[G])
}

/// The equality relation on a semiring is a semiring congruence.
theorem eq_relation_is_semiring_congruence[S: Semiring] {
    is_semiring_congruence(eq_relation[S])
} by {
    eq_relation_is_add_congruence[S]
    eq_relation_is_mul_congruence[S]
    is_semiring_congruence_intro(eq_relation[S])
}

/// The equality relation on a ring is a ring congruence.
theorem eq_relation_is_ring_congruence[R: Ring] {
    is_ring_congruence(eq_relation[R])
} by {
    eq_relation_is_add_congruence[R]
    eq_relation_is_mul_congruence[R]
    is_ring_congruence_intro(eq_relation[R])
}

/// A module congruence is in particular an additive congruence.
theorem is_module_congruence_is_add_congruence[R: Ring, M: AddCommGroup](
    src: Module[R, M], r: (M, M) -> Bool
) {
    is_module_congruence(src, r) implies is_add_congruence(r)
} by {
    if is_module_congruence(src, r) {
        is_module_congruence_is_add_group_congruence(src, r)
        is_add_group_congruence_is_add_congruence(r)
    }
}

/// A module congruence is in particular an additive-monoid congruence.
theorem is_module_congruence_is_add_monoid_congruence[R: Ring, M: AddCommGroup](
    src: Module[R, M], r: (M, M) -> Bool
) {
    is_module_congruence(src, r) implies is_add_monoid_congruence(r)
} by {
    if is_module_congruence(src, r) {
        is_module_congruence_is_add_congruence(src, r)
    }
}

/// The equality relation on a module carrier is a module congruence.
theorem eq_relation_is_module_congruence[R: Ring, M: AddCommGroup](
    src: Module[R, M]
) {
    is_module_congruence(src, eq_relation[M])
} by {
    eq_relation_is_add_group_congruence[M]
    forall(s: R) {
        eq_relation_respects_unary_op(src.smul(s))
        respects_unary_op(eq_relation[M], src.smul(s))
    }
    is_module_congruence_intro(src, eq_relation[M])
}

/// A module congruence is an equivalence relation.
theorem is_module_congruence_is_equivalence[R: Ring, M: AddCommGroup](
    src: Module[R, M], r: (M, M) -> Bool
) {
    is_module_congruence(src, r) implies is_equivalence(r)
} by {
    if is_module_congruence(src, r) {
        is_module_congruence_is_add_group_congruence(src, r)
        is_add_group_congruence_is_equivalence(r)
    }
}

/// An additive-group congruence respects addition.
theorem is_add_group_congruence_respects_add[A: AddGroup](r: (A, A) -> Bool) {
    is_add_group_congruence(r) implies respects_add(r)
} by {
    if is_add_group_congruence(r) {
        is_add_group_congruence_is_add_congruence(r)
        add_congruence_respects_add(r)
    }
}

/// A group congruence respects multiplication.
theorem is_group_congruence_respects_mul[G: Group](r: (G, G) -> Bool) {
    is_group_congruence(r) implies respects_mul(r)
} by {
    if is_group_congruence(r) {
        is_group_congruence_is_mul_congruence(r)
        mul_congruence_respects_mul(r)
    }
}

/// A semiring congruence respects addition.
theorem is_semiring_congruence_respects_add[S: Semiring](r: (S, S) -> Bool) {
    is_semiring_congruence(r) implies respects_add(r)
} by {
    if is_semiring_congruence(r) {
        is_semiring_congruence_is_add_congruence(r)
        add_congruence_respects_add(r)
    }
}

/// A semiring congruence respects multiplication.
theorem is_semiring_congruence_respects_mul[S: Semiring](r: (S, S) -> Bool) {
    is_semiring_congruence(r) implies respects_mul(r)
} by {
    if is_semiring_congruence(r) {
        is_semiring_congruence_is_mul_congruence(r)
        mul_congruence_respects_mul(r)
    }
}

/// A ring congruence respects addition.
theorem is_ring_congruence_respects_add[R: Ring](r: (R, R) -> Bool) {
    is_ring_congruence(r) implies respects_add(r)
} by {
    if is_ring_congruence(r) {
        is_ring_congruence_is_add_congruence(r)
        add_congruence_respects_add(r)
    }
}

/// A ring congruence respects multiplication.
theorem is_ring_congruence_respects_mul[R: Ring](r: (R, R) -> Bool) {
    is_ring_congruence(r) implies respects_mul(r)
} by {
    if is_ring_congruence(r) {
        is_ring_congruence_is_mul_congruence(r)
        mul_congruence_respects_mul(r)
    }
}

/// A monoid congruence respects multiplication.
theorem is_monoid_congruence_respects_mul[M: Monoid](r: (M, M) -> Bool) {
    is_monoid_congruence(r) implies respects_mul(r)
} by {
    if is_monoid_congruence(r) {
        is_monoid_congruence_is_mul_congruence(r)
        mul_congruence_respects_mul(r)
    }
}

/// An additive-monoid congruence respects addition.
theorem is_add_monoid_congruence_respects_add[A: AddMonoid](r: (A, A) -> Bool) {
    is_add_monoid_congruence(r) implies respects_add(r)
} by {
    if is_add_monoid_congruence(r) {
        is_add_monoid_congruence_is_add_congruence(r)
        add_congruence_respects_add(r)
    }
}

/// A module congruence respects addition.
theorem is_module_congruence_respects_add[R: Ring, M: AddCommGroup](
    src: Module[R, M], r: (M, M) -> Bool
) {
    is_module_congruence(src, r) implies respects_add(r)
} by {
    if is_module_congruence(src, r) {
        is_module_congruence_is_add_group_congruence(src, r)
        is_add_group_congruence_respects_add(r)
    }
}

/// A module congruence yields a unary congruence for scalar multiplication by every scalar.
theorem is_module_congruence_is_unary_congruence_smul[R: Ring, M: AddCommGroup](
    src: Module[R, M], r: (M, M) -> Bool, s: R
) {
    is_module_congruence(src, r) implies is_unary_congruence(r, src.smul(s))
} by {
    if is_module_congruence(src, r) {
        is_module_congruence_is_add_group_congruence(src, r)
        is_add_group_congruence_is_equivalence(r)
        is_module_congruence_respects_smul(src, r, s)
        is_unary_congruence(r, src.smul(s)) = (
            is_equivalence(r) and respects_unary_op(r, src.smul(s))
        )
        is_unary_congruence(r, src.smul(s))
    }
}

/// A module congruence respects negation.
theorem is_module_congruence_respects_neg[R: Ring, M: AddCommGroup](
    src: Module[R, M], r: (M, M) -> Bool
) {
    is_module_congruence(src, r) implies is_unary_congruence(r, M.neg)
} by {
    if is_module_congruence(src, r) {
        is_module_congruence_is_add_group_congruence(src, r)
        is_add_group_congruence_respects_neg(r)
    }
}

/// On a ring carrier, the additive part of a ring congruence respects negation.
theorem ring_add_congruence_respects_neg[R: Ring](r: (R, R) -> Bool) {
    is_add_congruence(r) implies respects_unary_op(r, R.neg)
} by {
    if is_add_congruence(r) {
        add_congruence_is_equivalence(r)
        add_congruence_respects_add(r)
        forall(x: R, y: R) {
            if r(x, y) {
                r(-x, -x)
                r(-x + x, -x + y)
                -x + x = R.0
                r(R.0, -x + y)
                r(-y, -y)
                r(R.0 + -y, (-x + y) + -y)
                R.0 + -y = -y
                (-x + y) + -y = -x + (y + -y)
                y + -y = R.0
                -x + (y + -y) = -x + R.0
                -x + R.0 = -x
                (-x + y) + -y = -x
                r(-y, -x)
                r(-x, -y)
                r(R.neg(x), R.neg(y))
            }
        }
    }
}

/// A ring congruence is in particular an additive-group congruence.
theorem is_ring_congruence_is_add_group_congruence[R: Ring](r: (R, R) -> Bool) {
    is_ring_congruence(r) implies is_add_group_congruence(r)
} by {
    if is_ring_congruence(r) {
        is_ring_congruence_is_add_congruence(r)
        ring_add_congruence_respects_neg(r)
        add_congruence_is_equivalence(r)
        is_unary_congruence(r, R.neg)
        is_add_group_congruence_iff(r)
        is_add_group_congruence(r)
    }
}

/// A ring congruence respects negation.
theorem is_ring_congruence_respects_neg[R: Ring](r: (R, R) -> Bool) {
    is_ring_congruence(r) implies is_unary_congruence(r, R.neg)
} by {
    if is_ring_congruence(r) {
        is_ring_congruence_is_add_group_congruence(r)
        is_add_group_congruence_respects_neg(r)
    }
}

/// An additive-group congruence directly respects the negation unary operation.
theorem is_add_group_congruence_respects_unary_op_neg[A: AddGroup](r: (A, A) -> Bool) {
    is_add_group_congruence(r) implies respects_unary_op(r, A.neg)
} by {
    if is_add_group_congruence(r) {
        is_add_group_congruence_respects_neg(r)
        is_unary_congruence(r, A.neg) = (
            is_equivalence(r) and respects_unary_op(r, A.neg)
        )
    }
}

/// A group congruence directly respects the inversion unary operation.
theorem is_group_congruence_respects_unary_op_inverse[G: Group](r: (G, G) -> Bool) {
    is_group_congruence(r) implies respects_unary_op(r, G.inverse)
} by {
    if is_group_congruence(r) {
        is_group_congruence_respects_inverse(r)
        is_unary_congruence(r, G.inverse) = (
            is_equivalence(r) and respects_unary_op(r, G.inverse)
        )
    }
}

/// A ring congruence directly respects the negation unary operation.
theorem is_ring_congruence_respects_unary_op_neg[R: Ring](r: (R, R) -> Bool) {
    is_ring_congruence(r) implies respects_unary_op(r, R.neg)
} by {
    if is_ring_congruence(r) {
        is_ring_congruence_respects_neg(r)
        is_unary_congruence(r, R.neg) = (
            is_equivalence(r) and respects_unary_op(r, R.neg)
        )
    }
}

/// A module congruence directly respects the carrier-level negation unary operation.
theorem is_module_congruence_respects_unary_op_neg[R: Ring, M: AddCommGroup](
    src: Module[R, M], r: (M, M) -> Bool
) {
    is_module_congruence(src, r) implies respects_unary_op(r, M.neg)
} by {
    if is_module_congruence(src, r) {
        is_module_congruence_respects_neg(src, r)
        is_unary_congruence(r, M.neg) = (
            is_equivalence(r) and respects_unary_op(r, M.neg)
        )
    }
}

/// An additive-group congruence directly respects the addition binary operation.
theorem is_add_group_congruence_respects_binary_op_add[A: AddGroup](r: (A, A) -> Bool) {
    is_add_group_congruence(r) implies respects_binary_op(r, A.add)
} by {
    if is_add_group_congruence(r) {
        is_add_group_congruence_respects_add(r)
        respects_add(r) = respects_binary_op(r, A.add)
    }
}

/// A group congruence directly respects the multiplication binary operation.
theorem is_group_congruence_respects_binary_op_mul[G: Group](r: (G, G) -> Bool) {
    is_group_congruence(r) implies respects_binary_op(r, G.mul)
} by {
    if is_group_congruence(r) {
        is_group_congruence_respects_mul(r)
        respects_mul(r) = respects_binary_op(r, G.mul)
    }
}

/// A ring congruence directly respects the addition binary operation.
theorem is_ring_congruence_respects_binary_op_add[R: Ring](r: (R, R) -> Bool) {
    is_ring_congruence(r) implies respects_binary_op(r, R.add)
} by {
    if is_ring_congruence(r) {
        is_ring_congruence_respects_add(r)
        respects_add(r) = respects_binary_op(r, R.add)
    }
}

/// A ring congruence directly respects the multiplication binary operation.
theorem is_ring_congruence_respects_binary_op_mul[R: Ring](r: (R, R) -> Bool) {
    is_ring_congruence(r) implies respects_binary_op(r, R.mul)
} by {
    if is_ring_congruence(r) {
        is_ring_congruence_respects_mul(r)
        respects_mul(r) = respects_binary_op(r, R.mul)
    }
}

/// A module congruence directly respects the carrier-level addition binary operation.
theorem is_module_congruence_respects_binary_op_add[R: Ring, M: AddCommGroup](
    src: Module[R, M], r: (M, M) -> Bool
) {
    is_module_congruence(src, r) implies respects_binary_op(r, M.add)
} by {
    if is_module_congruence(src, r) {
        is_module_congruence_respects_add(src, r)
        respects_add(r) = respects_binary_op(r, M.add)
    }
}

/// A monoid congruence directly respects the multiplication binary operation.
theorem is_monoid_congruence_respects_binary_op_mul[M: Monoid](r: (M, M) -> Bool) {
    is_monoid_congruence(r) implies respects_binary_op(r, M.mul)
} by {
    if is_monoid_congruence(r) {
        is_monoid_congruence_respects_mul(r)
        respects_mul(r) = respects_binary_op(r, M.mul)
    }
}

/// An additive-monoid congruence directly respects the addition binary operation.
theorem is_add_monoid_congruence_respects_binary_op_add[A: AddMonoid](r: (A, A) -> Bool) {
    is_add_monoid_congruence(r) implies respects_binary_op(r, A.add)
} by {
    if is_add_monoid_congruence(r) {
        is_add_monoid_congruence_respects_add(r)
        respects_add(r) = respects_binary_op(r, A.add)
    }
}

/// A semiring congruence directly respects the addition binary operation.
theorem is_semiring_congruence_respects_binary_op_add[S: Semiring](r: (S, S) -> Bool) {
    is_semiring_congruence(r) implies respects_binary_op(r, S.add)
} by {
    if is_semiring_congruence(r) {
        is_semiring_congruence_respects_add(r)
        respects_add(r) = respects_binary_op(r, S.add)
    }
}

/// A semiring congruence directly respects the multiplication binary operation.
theorem is_semiring_congruence_respects_binary_op_mul[S: Semiring](r: (S, S) -> Bool) {
    is_semiring_congruence(r) implies respects_binary_op(r, S.mul)
} by {
    if is_semiring_congruence(r) {
        is_semiring_congruence_respects_mul(r)
        respects_mul(r) = respects_binary_op(r, S.mul)
    }
}

/// An additive-group congruence is a unary congruence for negation.
theorem is_add_group_congruence_is_unary_congruence_neg[A: AddGroup](r: (A, A) -> Bool) {
    is_add_group_congruence(r) implies is_unary_congruence(r, A.neg)
} by {
    if is_add_group_congruence(r) {
        is_add_group_congruence_is_equivalence(r)
        is_add_group_congruence_respects_unary_op_neg(r)
        is_unary_congruence(r, A.neg) = (
            is_equivalence(r) and respects_unary_op(r, A.neg)
        )
        is_unary_congruence(r, A.neg)
    }
}

/// A group congruence is a unary congruence for inversion.
theorem is_group_congruence_is_unary_congruence_inverse[G: Group](r: (G, G) -> Bool) {
    is_group_congruence(r) implies is_unary_congruence(r, G.inverse)
} by {
    if is_group_congruence(r) {
        is_group_congruence_is_equivalence(r)
        is_group_congruence_respects_unary_op_inverse(r)
        is_unary_congruence(r, G.inverse) = (
            is_equivalence(r) and respects_unary_op(r, G.inverse)
        )
        is_unary_congruence(r, G.inverse)
    }
}

/// A ring congruence is a unary congruence for negation.
theorem is_ring_congruence_is_unary_congruence_neg[R: Ring](r: (R, R) -> Bool) {
    is_ring_congruence(r) implies is_unary_congruence(r, R.neg)
} by {
    if is_ring_congruence(r) {
        is_ring_congruence_is_equivalence(r)
        is_ring_congruence_respects_unary_op_neg(r)
        is_unary_congruence(r, R.neg) = (
            is_equivalence(r) and respects_unary_op(r, R.neg)
        )
        is_unary_congruence(r, R.neg)
    }
}

/// A module congruence is a unary congruence for carrier-level negation.
theorem is_module_congruence_is_unary_congruence_neg[R: Ring, M: AddCommGroup](
    src: Module[R, M], r: (M, M) -> Bool
) {
    is_module_congruence(src, r) implies is_unary_congruence(r, M.neg)
} by {
    if is_module_congruence(src, r) {
        is_module_congruence_is_equivalence(src, r)
        is_module_congruence_respects_unary_op_neg(src, r)
        is_unary_congruence(r, M.neg)
    }
}

/// A monoid congruence is a binary congruence for multiplication.
theorem is_monoid_congruence_is_binary_congruence_mul[M: Monoid](r: (M, M) -> Bool) {
    is_monoid_congruence(r) implies is_binary_congruence(r, M.mul)
} by {
    if is_monoid_congruence(r) {
        is_monoid_congruence_is_equivalence(r)
        is_monoid_congruence_respects_binary_op_mul(r)
        is_binary_congruence(r, M.mul) = (
            is_equivalence(r) and respects_binary_op(r, M.mul)
        )
        is_binary_congruence(r, M.mul)
    }
}

/// An additive-monoid congruence is a binary congruence for addition.
theorem is_add_monoid_congruence_is_binary_congruence_add[A: AddMonoid](r: (A, A) -> Bool) {
    is_add_monoid_congruence(r) implies is_binary_congruence(r, A.add)
} by {
    if is_add_monoid_congruence(r) {
        is_add_monoid_congruence_is_equivalence(r)
        is_add_monoid_congruence_respects_binary_op_add(r)
        is_binary_congruence(r, A.add) = (
            is_equivalence(r) and respects_binary_op(r, A.add)
        )
        is_binary_congruence(r, A.add)
    }
}

/// A group congruence is a binary congruence for multiplication.
theorem is_group_congruence_is_binary_congruence_mul[G: Group](r: (G, G) -> Bool) {
    is_group_congruence(r) implies is_binary_congruence(r, G.mul)
} by {
    if is_group_congruence(r) {
        is_group_congruence_is_equivalence(r)
        is_group_congruence_respects_binary_op_mul(r)
        is_binary_congruence(r, G.mul) = (
            is_equivalence(r) and respects_binary_op(r, G.mul)
        )
        is_binary_congruence(r, G.mul)
    }
}

/// An additive-group congruence is a binary congruence for addition.
theorem is_add_group_congruence_is_binary_congruence_add[A: AddGroup](r: (A, A) -> Bool) {
    is_add_group_congruence(r) implies is_binary_congruence(r, A.add)
} by {
    if is_add_group_congruence(r) {
        is_add_group_congruence_is_equivalence(r)
        is_add_group_congruence_respects_binary_op_add(r)
        is_binary_congruence(r, A.add) = (
            is_equivalence(r) and respects_binary_op(r, A.add)
        )
        is_binary_congruence(r, A.add)
    }
}

/// A semiring congruence is a binary congruence for addition.
theorem is_semiring_congruence_is_binary_congruence_add[S: Semiring](r: (S, S) -> Bool) {
    is_semiring_congruence(r) implies is_binary_congruence(r, S.add)
} by {
    if is_semiring_congruence(r) {
        is_semiring_congruence_is_equivalence(r)
        is_semiring_congruence_respects_binary_op_add(r)
        is_binary_congruence(r, S.add) = (
            is_equivalence(r) and respects_binary_op(r, S.add)
        )
        is_binary_congruence(r, S.add)
    }
}

/// A semiring congruence is a binary congruence for multiplication.
theorem is_semiring_congruence_is_binary_congruence_mul[S: Semiring](r: (S, S) -> Bool) {
    is_semiring_congruence(r) implies is_binary_congruence(r, S.mul)
} by {
    if is_semiring_congruence(r) {
        is_semiring_congruence_is_equivalence(r)
        is_semiring_congruence_respects_binary_op_mul(r)
        is_binary_congruence(r, S.mul) = (
            is_equivalence(r) and respects_binary_op(r, S.mul)
        )
        is_binary_congruence(r, S.mul)
    }
}

/// A ring congruence is a binary congruence for addition.
theorem is_ring_congruence_is_binary_congruence_add[R: Ring](r: (R, R) -> Bool) {
    is_ring_congruence(r) implies is_binary_congruence(r, R.add)
} by {
    if is_ring_congruence(r) {
        is_ring_congruence_is_equivalence(r)
        is_ring_congruence_respects_binary_op_add(r)
        is_binary_congruence(r, R.add) = (
            is_equivalence(r) and respects_binary_op(r, R.add)
        )
        is_binary_congruence(r, R.add)
    }
}

/// A ring congruence is a binary congruence for multiplication.
theorem is_ring_congruence_is_binary_congruence_mul[R: Ring](r: (R, R) -> Bool) {
    is_ring_congruence(r) implies is_binary_congruence(r, R.mul)
} by {
    if is_ring_congruence(r) {
        is_ring_congruence_is_equivalence(r)
        is_ring_congruence_respects_binary_op_mul(r)
        is_binary_congruence(r, R.mul) = (
            is_equivalence(r) and respects_binary_op(r, R.mul)
        )
        is_binary_congruence(r, R.mul)
    }
}

/// A module congruence is a binary congruence for carrier-level addition.
theorem is_module_congruence_is_binary_congruence_add[R: Ring, M: AddCommGroup](
    src: Module[R, M], r: (M, M) -> Bool
) {
    is_module_congruence(src, r) implies is_binary_congruence(r, M.add)
} by {
    if is_module_congruence(src, r) {
        is_module_congruence_is_equivalence(src, r)
        is_module_congruence_respects_binary_op_add(src, r)
        is_binary_congruence(r, M.add)
    }
}

/// The kernel of a monoid homomorphism is a binary congruence for multiplication.
theorem monoid_hom_kernel_is_binary_congruence_mul[M: Monoid, N: Monoid](f: MonoidHom[M, N]) {
    is_binary_congruence(kernel_relation(f.hom), M.mul)
} by {
    monoid_hom_kernel_is_monoid_congruence(f)
    is_monoid_congruence_is_binary_congruence_mul(kernel_relation(f.hom))
}

/// The kernel of an additive-monoid homomorphism is a binary congruence for addition.
theorem add_monoid_hom_kernel_is_binary_congruence_add[A: AddMonoid, B: AddMonoid](f: AddMonoidHom[A, B]) {
    is_binary_congruence(kernel_relation(f.hom), A.add)
} by {
    add_monoid_hom_kernel_is_add_monoid_congruence(f)
    is_add_monoid_congruence_is_binary_congruence_add(kernel_relation(f.hom))
}

/// The kernel of a group homomorphism is a binary congruence for multiplication.
theorem group_hom_kernel_is_binary_congruence_mul[G: Group, H: Group](f: GroupHom[G, H]) {
    is_binary_congruence(kernel_relation(f.hom), G.mul)
} by {
    group_hom_kernel_is_group_congruence(f)
    is_group_congruence_is_binary_congruence_mul(kernel_relation(f.hom))
}

/// The kernel of an additive-group homomorphism is a binary congruence for addition.
theorem add_group_hom_kernel_is_binary_congruence_add[A: AddGroup, B: AddGroup](f: AddGroupHom[A, B]) {
    is_binary_congruence(kernel_relation(f.hom), A.add)
} by {
    add_group_hom_kernel_is_add_group_congruence(f)
    is_add_group_congruence_is_binary_congruence_add(kernel_relation(f.hom))
}

/// The kernel of a semiring homomorphism is a binary congruence for addition.
theorem semiring_hom_kernel_is_binary_congruence_add[S: Semiring, T: Semiring](f: SemiringHom[S, T]) {
    is_binary_congruence(kernel_relation(f.hom), S.add)
} by {
    semiring_hom_kernel_is_semiring_congruence(f)
    is_semiring_congruence_is_binary_congruence_add(kernel_relation(f.hom))
}

/// The kernel of a semiring homomorphism is a binary congruence for multiplication.
theorem semiring_hom_kernel_is_binary_congruence_mul[S: Semiring, T: Semiring](f: SemiringHom[S, T]) {
    is_binary_congruence(kernel_relation(f.hom), S.mul)
} by {
    semiring_hom_kernel_is_semiring_congruence(f)
    is_semiring_congruence_is_binary_congruence_mul(kernel_relation(f.hom))
}

/// The kernel of a ring homomorphism is a binary congruence for addition.
theorem ring_hom_kernel_is_binary_congruence_add[R: Ring, S: Ring](f: RingHom[R, S]) {
    is_binary_congruence(kernel_relation(f.hom), R.add)
} by {
    ring_hom_kernel_is_ring_congruence(f)
    is_ring_congruence_is_binary_congruence_add(kernel_relation(f.hom))
}

/// The kernel of a ring homomorphism is a binary congruence for multiplication.
theorem ring_hom_kernel_is_binary_congruence_mul[R: Ring, S: Ring](f: RingHom[R, S]) {
    is_binary_congruence(kernel_relation(f.hom), R.mul)
} by {
    ring_hom_kernel_is_ring_congruence(f)
    is_ring_congruence_is_binary_congruence_mul(kernel_relation(f.hom))
}

/// The kernel of a linear map is a binary congruence for carrier-level addition.
theorem linear_map_kernel_is_binary_congruence_add[R: Ring, M: AddCommGroup, N: AddCommGroup](
    src: Module[R, M], dst: Module[R, N], f: M -> N
) {
    is_linear_map(src, dst, f) implies is_binary_congruence(kernel_relation(f), M.add)
} by {
    if is_linear_map(src, dst, f) {
        linear_map_kernel_is_module_congruence(src, dst, f)
        is_module_congruence_is_binary_congruence_add(src, kernel_relation(f))
    }
}

/// The kernel of an additive-group homomorphism is a unary congruence for negation.
theorem add_group_hom_kernel_is_unary_congruence_neg[A: AddGroup, B: AddGroup](f: AddGroupHom[A, B]) {
    is_unary_congruence(kernel_relation(f.hom), A.neg)
} by {
    add_group_hom_kernel_is_add_group_congruence(f)
    is_add_group_congruence_is_unary_congruence_neg(kernel_relation(f.hom))
}

/// The kernel of a group homomorphism is a unary congruence for inversion.
theorem group_hom_kernel_is_unary_congruence_inverse[G: Group, H: Group](f: GroupHom[G, H]) {
    is_unary_congruence(kernel_relation(f.hom), G.inverse)
} by {
    group_hom_kernel_is_group_congruence(f)
    is_group_congruence_is_unary_congruence_inverse(kernel_relation(f.hom))
}

/// The kernel of a ring homomorphism is a unary congruence for negation.
theorem ring_hom_kernel_is_unary_congruence_neg[R: Ring, S: Ring](f: RingHom[R, S]) {
    is_unary_congruence(kernel_relation(f.hom), R.neg)
} by {
    ring_hom_kernel_is_ring_congruence(f)
    is_ring_congruence_is_unary_congruence_neg(kernel_relation(f.hom))
}

/// The kernel of a linear map is a unary congruence for carrier-level negation.
theorem linear_map_kernel_is_unary_congruence_neg[R: Ring, M: AddCommGroup, N: AddCommGroup](
    src: Module[R, M], dst: Module[R, N], f: M -> N
) {
    is_linear_map(src, dst, f) implies is_unary_congruence(kernel_relation(f), M.neg)
} by {
    if is_linear_map(src, dst, f) {
        linear_map_kernel_is_module_congruence(src, dst, f)
        is_module_congruence_is_unary_congruence_neg(src, kernel_relation(f))
    }
}

/// The kernel of a linear map is a unary congruence for scalar multiplication by every scalar.
theorem linear_map_kernel_is_unary_congruence_smul[R: Ring, M: AddCommGroup, N: AddCommGroup](
    src: Module[R, M], dst: Module[R, N], f: M -> N, s: R
) {
    is_linear_map(src, dst, f) implies is_unary_congruence(kernel_relation(f), src.smul(s))
} by {
    if is_linear_map(src, dst, f) {
        linear_map_kernel_is_module_congruence(src, dst, f)
        is_module_congruence_is_unary_congruence_smul(src, kernel_relation(f), s)
    }
}

/// The kernel of an additive-group homomorphism respects carrier-level negation.
theorem add_group_hom_kernel_respects_unary_op_neg[A: AddGroup, B: AddGroup](f: AddGroupHom[A, B]) {
    respects_unary_op(kernel_relation(f.hom), A.neg)
} by {
    add_group_hom_kernel_is_unary_congruence_neg(f)
}

/// The kernel of a group homomorphism respects carrier-level inversion.
theorem group_hom_kernel_respects_unary_op_inverse[G: Group, H: Group](f: GroupHom[G, H]) {
    respects_unary_op(kernel_relation(f.hom), G.inverse)
} by {
    group_hom_kernel_is_unary_congruence_inverse(f)
}

/// The kernel of a ring homomorphism respects carrier-level negation.
theorem ring_hom_kernel_respects_unary_op_neg[R: Ring, S: Ring](f: RingHom[R, S]) {
    respects_unary_op(kernel_relation(f.hom), R.neg)
} by {
    ring_hom_kernel_is_unary_congruence_neg(f)
}

/// The kernel of a linear map respects carrier-level negation.
theorem linear_map_kernel_respects_unary_op_neg[R: Ring, M: AddCommGroup, N: AddCommGroup](
    src: Module[R, M], dst: Module[R, N], f: M -> N
) {
    is_linear_map(src, dst, f) implies respects_unary_op(kernel_relation(f), M.neg)
} by {
    if is_linear_map(src, dst, f) {
        linear_map_kernel_is_unary_congruence_neg(src, dst, f)
        is_unary_congruence(kernel_relation(f), M.neg) = (
            is_equivalence(kernel_relation(f)) and respects_unary_op(kernel_relation(f), M.neg)
        )
    }
}
