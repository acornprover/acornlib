from comm_ring import CommRing
from algebra.field.field import Field
from algebra.ring.ideal import MaximalIdeal, is_ideal, is_ideal_absorb, is_ideal_zero,
    is_maximal_ideal, is_maximal_ideal_is_ideal, is_maximal_ideal_proper,
    is_maximal_ideal_from_constraints, is_maximal_ideal_witness,
    ideal_proper_constraint, ideal_subset, zero_ideal
from algebra.field.field_ideal import zero_ideal_is_maximal
from algebra.comm_ring_unit import non_units, non_units_not_one,
    proper_ideal_subset_non_units

/// True if `R` is a local ring witnessed by `m`: `m` is a maximal ideal,
/// and every maximal ideal coincides with `m`.
define is_local_ring[R: CommRing](m: R -> Bool) -> Bool {
    is_maximal_ideal(m)
    and forall(other: R -> Bool) {
        is_maximal_ideal(other) implies other = m
    }
}

/// The local-ring predicate unfolds to maximality plus uniqueness of maximal ideals.
theorem is_local_ring_unfold[R: CommRing](m: R -> Bool) {
    is_local_ring(m) = (is_maximal_ideal(m) and forall(other: R -> Bool) {
        is_maximal_ideal(other) implies other = m
    })
}

/// Maximality plus uniqueness of maximal ideals establishes the local-ring predicate.
theorem is_local_ring_from_unique_maximal[R: CommRing](m: R -> Bool) {
    is_maximal_ideal(m) and (forall(other: R -> Bool) {
        is_maximal_ideal(other) implies other = m
    }) implies is_local_ring(m)
} by {
    if is_maximal_ideal(m) and forall(other: R -> Bool) {
        is_maximal_ideal(other) implies other = m
    } {
        is_local_ring_unfold(m)
        is_local_ring(m)
    }
}

/// In a local ring, the witness is itself a maximal ideal.
theorem local_ring_witness_is_maximal[R: CommRing](m: R -> Bool) {
    is_local_ring(m) implies is_maximal_ideal(m)
} by {
    if is_local_ring(m) {
        is_local_ring(m) = (is_maximal_ideal(m)
            and forall(other: R -> Bool) {
                is_maximal_ideal(other) implies other = m
            })
    }
}

/// In a local ring, every maximal ideal equals the witnessing one.
theorem local_ring_maximal_unique[R: CommRing](m: R -> Bool, other: R -> Bool) {
    is_local_ring(m) and is_maximal_ideal(other) implies other = m
} by {
    if is_local_ring(m) and is_maximal_ideal(other) {
        is_local_ring(m) = (is_maximal_ideal(m)
            and forall(n: R -> Bool) {
                is_maximal_ideal(n) implies n = m
            })
    }
}

/// Any two local-ring witnesses coincide.
theorem local_ring_witness_eq_of_is_local_ring[R: CommRing](m: R -> Bool, n: R -> Bool) {
    is_local_ring(m) and is_local_ring(n) implies n = m
} by {
    if is_local_ring(m) and is_local_ring(n) {
        local_ring_witness_is_maximal(n)
        is_maximal_ideal(n)
        local_ring_maximal_unique(m, n)
    }
}

/// In a field, every maximal ideal equals the zero ideal.
theorem field_maximal_ideal_eq_zero_ideal[F: Field](other: F -> Bool) {
    is_maximal_ideal(other) implies other = zero_ideal[F]
} by {
    if is_maximal_ideal(other) {
        is_maximal_ideal_is_ideal(other)
        is_maximal_ideal_proper(other)

        is_ideal_zero(other)
        forall(x: F) {
            if other(x) {
                if x != F.0 {
                    is_ideal_absorb(other, x.inverse, x)
                    x.inverse * x = F.1
                    false
                }
                zero_ideal[F](x)
            }
            if zero_ideal[F](x) {
                x = F.0
                other(x)
            }
            other(x) = zero_ideal[F](x)
        }
        other = zero_ideal[F]
    }
}

/// A local ring of a commutative ring, bundled with its witnessing maximal ideal.
structure LocalRing[R: CommRing] {
    /// The unique maximal ideal of this local ring, as a membership predicate.
    maximal: R -> Bool
} constraint {
    is_local_ring(maximal)
}

/// The witnessing predicate of a bundled local ring is a maximal ideal.
theorem local_ring_maximal_is_maximal_ideal[R: CommRing](lr: LocalRing[R]) {
    is_maximal_ideal(lr.maximal)
} by {
    local_ring_witness_is_maximal(lr.maximal)
}

/// In a bundled local ring, every maximal ideal equals the bundled witness.
theorem local_ring_unique_maximal[R: CommRing](lr: LocalRing[R], other: R -> Bool) {
    is_maximal_ideal(other) implies other = lr.maximal
} by {
    if is_maximal_ideal(other) {
        is_local_ring(lr.maximal)
        local_ring_maximal_unique(lr.maximal, other)
    }
}

/// The bundled maximal ideal underlying a local ring.
let local_ring_as_maximal_ideal[R: CommRing](lr: LocalRing[R]) -> result: MaximalIdeal[R] satisfy {
    MaximalIdeal.new(lr.maximal) = Option.some(result)
} by {
    local_ring_maximal_is_maximal_ideal(lr)
}

/// Bundled local ring extensionality from equality of the maximal witness.
theorem local_ring_ext[R: CommRing](a: LocalRing[R], b: LocalRing[R]) {
    a.maximal = b.maximal implies a = b
}

/// Bundled local ring extensionality from pointwise equality of maximal-witness membership.
theorem local_ring_ext_at[R: CommRing](a: LocalRing[R], b: LocalRing[R]) {
    (forall(x: R) { a.maximal(x) = b.maximal(x) }) implies a = b
} by {
    if forall(x: R) { a.maximal(x) = b.maximal(x) } {
        a.maximal = b.maximal
        local_ring_ext(a, b)
    }
}

attributes LocalRing[R: CommRing] {
    /// Bundled local ring extensionality from equality of the maximal witness.
    let ext = local_ring_ext[R]

    /// The bundled maximal ideal obtained from the local ring structure.
    let as_maximal_ideal: LocalRing[R] -> MaximalIdeal[R] = local_ring_as_maximal_ideal[R]
}

/// Membership in the underlying maximal ideal of a local ring coincides with membership in the local ring's maximal predicate.
theorem local_ring_as_maximal_ideal_contains_eq[R: CommRing](lr: LocalRing[R], x: R) {
    lr.as_maximal_ideal.contains(x) = lr.maximal(x)
}

/// If the non-units form an ideal, then this ideal is proper.
theorem non_units_ideal_proper[R: CommRing] {
    ideal_proper_constraint(non_units[R])
} by {
    non_units_not_one[R]
}

/// If the non-units form an ideal, then this ideal is a maximal ideal.
theorem non_units_ideal_is_maximal[R: CommRing] {
    is_ideal(non_units[R]) implies is_maximal_ideal(non_units[R])
} by {
    if is_ideal(non_units[R]) {
        non_units_ideal_proper[R]
        forall(other: R -> Bool) {
            if is_ideal(other) and ideal_subset(non_units[R], other)
                and ideal_proper_constraint(other) {
                proper_ideal_subset_non_units(other)
                ideal_subset(other, non_units[R])
            }
        }
        is_maximal_ideal_from_constraints(non_units[R])
    }
}

/// If the non-units form an ideal, then every maximal ideal coincides with `non_units`.
theorem non_units_ideal_maximal_unique[R: CommRing](other: R -> Bool) {
    is_ideal(non_units[R]) and is_maximal_ideal(other) implies other = non_units[R]
} by {
    if is_ideal(non_units[R]) and is_maximal_ideal(other) {
        is_maximal_ideal_is_ideal(other)
        is_maximal_ideal_proper(other)
        ideal_proper_constraint(other)
        proper_ideal_subset_non_units(other)
        non_units_ideal_proper[R]
        is_maximal_ideal_witness(other, non_units[R])
        forall(x: R) {
            if other(x) {
                ideal_subset(other, non_units[R]) = forall(y: R) {
                    other(y) implies non_units[R](y)
                }
                non_units[R](x)
            }
            if non_units[R](x) {
                ideal_subset(non_units[R], other) = forall(y: R) {
                    non_units[R](y) implies other(y)
                }
                other(x)
            }
            other(x) = non_units[R](x)
        }
    }
}

/// If the non-units form an ideal, then the ring is a local ring witnessed by `non_units`.
theorem non_units_ideal_implies_local_ring[R: CommRing] {
    is_ideal(non_units[R]) implies is_local_ring(non_units[R])
} by {
    if is_ideal(non_units[R]) {
        non_units_ideal_is_maximal[R]
        let m: R -> Bool = non_units[R]
        is_maximal_ideal(m)
        forall(n: R -> Bool) {
            if is_maximal_ideal(n) {
                non_units_ideal_maximal_unique(n)
                n = m
            }
        }
        is_local_ring(non_units[R])
    }
}

/// In a field, the zero ideal witnesses that the field is a local ring.
theorem field_is_local_ring[F: Field] {
    is_local_ring(zero_ideal[F])
} by {
    zero_ideal_is_maximal[F]

    forall(other: F -> Bool) {
        if is_maximal_ideal(other) {
            field_maximal_ideal_eq_zero_ideal[F](other)
            other = zero_ideal[F]
        }
    }
}
