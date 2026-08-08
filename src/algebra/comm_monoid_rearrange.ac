from algebra.comm_monoid import CommMonoid

/// Swapping the inner two factors of a product of two products.
///
/// The multiplicative twin of `add_swap_inner`. It appears whenever two multiplicative
/// structures are combined, and the associativity and commutativity steps are tedious enough to
/// be worth naming once. `src/nat_mul_rearrange.ac` had this for the naturals only; stated over
/// a commutative monoid it serves any multiplicative structure.
theorem mul_swap_inner[M: CommMonoid](a: M, b: M, c: M, d: M) {
    (a * b) * (c * d) = (a * c) * (b * d)
} by {
    (a * (b * (c * d)) = (a * b) * (c * d))
    (b * (c * d) = (b * c) * d)
    (b * c = c * b)
    ((b * c) * d = (c * b) * d)
    (c * (b * d) = (c * b) * d)
    b * (c * d) = c * (b * d)
    a * (b * (c * d)) = a * (c * (b * d))
    (a * (c * (b * d)) = (a * c) * (b * d))
}
