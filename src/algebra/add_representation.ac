from algebra.group import Group
from algebra.ring.ring import Ring
from algebra.add_comm_group import AddCommGroup
from algebra.add_group import AddGroupHom, is_add_group_hom, left_cancel,
    function_eq_transport_add_group_hom
from algebra.group_action import MulAction, is_mul_action, action_identity_constraint,
    action_mul_constraint
from algebra.module.module import Module, module_smul_add_right
from data.basic.set import Set
from data.basic.functions import identity_fn, compose, is_injective_fn, is_surjective_fn,
    is_bijection_fn, function_extensionality

/// True if every element of G acts on M by an additive homomorphism.
define action_is_add_hom[G: Group, M: AddCommGroup](act: (G, M) -> M) -> Bool {
    forall(g: G) {
        is_add_group_hom(function(x: M) { act(g, x) })
    }
}

/// True if a binary operation is a left action of a group on an additive commutative group
/// by additive homomorphisms.
define is_add_representation[G: Group, M: AddCommGroup](act: (G, M) -> M) -> Bool {
    is_mul_action(act) and action_is_add_hom(act)
}

/// A linear representation of a group on an additive commutative group:
/// every group element acts by an additive homomorphism.
structure AddRepresentation[G: Group, M: AddCommGroup] {
    /// The action map.
    act: (G, M) -> M
} constraint {
    is_add_representation(act)
}

/// The underlying multiplicative action of a group representation.
let add_rep_mul_action[G: Group, M: AddCommGroup](r: AddRepresentation[G, M]) -> result: MulAction[G, M] satisfy {
    MulAction[G, M].new(r.act) = Option.some(result)
} by {
    is_mul_action(r.act)
}

/// The action map of the underlying multiplicative action equals the representation action.
theorem add_rep_mul_action_act[G: Group, M: AddCommGroup](r: AddRepresentation[G, M]) {
    add_rep_mul_action(r).act = r.act
}

/// The identity element acts trivially.
theorem add_rep_one[G: Group, M: AddCommGroup](r: AddRepresentation[G, M], x: M) {
    r.act(G.1, x) = x
} by {
    is_mul_action(r.act)
}

/// Multiplication in the group is composition of actions.
theorem add_rep_mul[G: Group, M: AddCommGroup](r: AddRepresentation[G, M], g: G, h: G, x: M) {
    r.act(g * h, x) = r.act(g, r.act(h, x))
} by {
    is_mul_action(r.act)
    action_mul_constraint(r.act)
    action_mul_constraint(r.act) = forall(k: G, l: G, y: M) {
        r.act(k * l, y) = r.act(k, r.act(l, y))
    }
}

/// Each group element acts by an additive homomorphism.
theorem add_rep_act_is_add_hom[G: Group, M: AddCommGroup](r: AddRepresentation[G, M], g: G) {
    is_add_group_hom(function(x: M) { r.act(g, x) })
} by {
    action_is_add_hom(r.act)
}

/// The action distributes over addition.
theorem add_rep_act_add[G: Group, M: AddCommGroup](r: AddRepresentation[G, M], g: G, x: M, y: M) {
    r.act(g, x + y) = r.act(g, x) + r.act(g, y)
} by {
    add_rep_act_is_add_hom(r, g)
    let f: M -> M = function(z: M) { r.act(g, z) }
    is_add_group_hom(f) = forall(a: M, b: M) {
        f(a + b) = f(a) + f(b)
    }
    f(x + y) = f(x) + f(y)
}

/// The action sends zero to zero.
theorem add_rep_act_zero[G: Group, M: AddCommGroup](r: AddRepresentation[G, M], g: G) {
    r.act(g, M.0) = M.0
} by {
    add_rep_act_add(r, g, M.0, M.0)
    r.act(g, M.0 + M.0) = r.act(g, M.0) + r.act(g, M.0)
    M.0 + M.0 = M.0
    r.act(g, M.0) = r.act(g, M.0) + r.act(g, M.0)
    r.act(g, M.0) + M.0 = r.act(g, M.0) + r.act(g, M.0)
}

/// The action commutes with negation.
theorem add_rep_act_neg[G: Group, M: AddCommGroup](r: AddRepresentation[G, M], g: G, x: M) {
    r.act(g, -x) = -r.act(g, x)
} by {
    add_rep_act_add(r, g, x, -x)
    add_rep_act_zero(r, g)
    left_cancel(r.act(g, x), r.act(g, -x), -r.act(g, x))
}

/// Acting by the inverse on the left cancels an action.
theorem add_rep_inv_apply_apply[G: Group, M: AddCommGroup](r: AddRepresentation[G, M], g: G, x: M) {
    r.act(g.inverse, r.act(g, x)) = x
} by {
    add_rep_mul(r, g.inverse, g, x)
    add_rep_one(r, x)
}

/// Acting by the inverse on the right cancels an action.
theorem add_rep_apply_inv_apply[G: Group, M: AddCommGroup](r: AddRepresentation[G, M], g: G, x: M) {
    r.act(g, r.act(g.inverse, x)) = x
} by {
    add_rep_mul(r, g, g.inverse, x)
    add_rep_one(r, x)
}

/// The self-map induced by an element of an additive representation.
define add_rep_action_map[G: Group, M: AddCommGroup](r: AddRepresentation[G, M], g: G, x: M) -> M {
    r.act(g, x)
}

/// The induced action map evaluates to the representation action.
theorem add_rep_action_map_apply[G: Group, M: AddCommGroup](r: AddRepresentation[G, M], g: G, x: M) {
    add_rep_action_map(r, g, x) = r.act(g, x)
}

/// The identity element induces the identity self-map.
theorem add_rep_action_map_one[G: Group, M: AddCommGroup](r: AddRepresentation[G, M]) {
    add_rep_action_map(r, G.1) = identity_fn[M]
} by {
    forall(x: M) {
        add_rep_one(r, x)
        add_rep_action_map(r, G.1, x) = identity_fn[M](x)
    }
    function_extensionality(add_rep_action_map(r, G.1), identity_fn[M])
}

/// Multiplication in the group induces composition of additive action maps.
theorem add_rep_action_map_mul[G: Group, M: AddCommGroup](r: AddRepresentation[G, M], g: G, h: G) {
    add_rep_action_map(r, g * h) = compose(add_rep_action_map(r, g), add_rep_action_map(r, h))
} by {
    forall(x: M) {
        add_rep_mul(r, g, h, x)
        add_rep_action_map(r, g * h, x) = compose(add_rep_action_map(r, g), add_rep_action_map(r, h), x)
    }
    function_extensionality(add_rep_action_map(r, g * h), compose(add_rep_action_map(r, g), add_rep_action_map(r, h)))
}

/// The inverse element gives a left inverse for the induced additive action map.
theorem add_rep_action_map_inverse_apply_apply[G: Group, M: AddCommGroup](
    r: AddRepresentation[G, M], g: G, x: M
) {
    add_rep_action_map(r, g.inverse, add_rep_action_map(r, g, x)) = x
} by {
    add_rep_action_map_apply(r, g, x)
    add_rep_action_map_apply(r, g.inverse, add_rep_action_map(r, g, x))
    add_rep_inv_apply_apply(r, g, x)
}

/// The inverse element gives a right inverse for the induced additive action map.
theorem add_rep_action_map_apply_inverse_apply[G: Group, M: AddCommGroup](
    r: AddRepresentation[G, M], g: G, x: M
) {
    add_rep_action_map(r, g, add_rep_action_map(r, g.inverse, x)) = x
} by {
    add_rep_action_map_apply(r, g.inverse, x)
    add_rep_action_map_apply(r, g, add_rep_action_map(r, g.inverse, x))
    add_rep_apply_inv_apply(r, g, x)
}

/// Each induced additive action map is injective.
theorem add_rep_action_map_is_injective[G: Group, M: AddCommGroup](r: AddRepresentation[G, M], g: G) {
    is_injective_fn(add_rep_action_map(r, g))
} by {
    forall(x: M, y: M) {
        if add_rep_action_map(r, g, x) = add_rep_action_map(r, g, y) {
            add_rep_action_map_inverse_apply_apply(r, g, x)
            add_rep_action_map_inverse_apply_apply(r, g, y)
            x = y
        }
    }
}

/// Each induced additive action map is surjective.
theorem add_rep_action_map_is_surjective[G: Group, M: AddCommGroup](r: AddRepresentation[G, M], g: G) {
    is_surjective_fn(add_rep_action_map(r, g))
} by {
    forall(y: M) {
        let x = add_rep_action_map(r, g.inverse, y)
        add_rep_action_map_apply_inverse_apply(r, g, y)
        exists(z: M) {
            z = x and add_rep_action_map(r, g, z) = y
        }
    }
}

/// Each induced additive action map is a bijection.
theorem add_rep_action_map_is_bijection[G: Group, M: AddCommGroup](r: AddRepresentation[G, M], g: G) {
    is_bijection_fn(add_rep_action_map(r, g))
} by {
    add_rep_action_map_is_injective(r, g)
    add_rep_action_map_is_surjective(r, g)
}

/// The action map at a fixed group element is an additive group homomorphism.
let add_rep_action_hom[G: Group, M: AddCommGroup](r: AddRepresentation[G, M], g: G) -> result: AddGroupHom[M, M] satisfy {
    AddGroupHom.new(add_rep_action_map(r, g)) = Option.some(result)
} by {
    add_rep_act_is_add_hom(r, g)
    function_eq_transport_add_group_hom(function(x: M) { r.act(g, x) }, add_rep_action_map(r, g))
    is_add_group_hom(add_rep_action_map(r, g))
}

/// The bundled additive action homomorphism has the expected underlying map.
theorem add_rep_action_hom_hom[G: Group, M: AddCommGroup](r: AddRepresentation[G, M], g: G) {
    add_rep_action_hom(r, g).hom = add_rep_action_map(r, g)
} by {
    AddGroupHom.new(add_rep_action_map(r, g)) = Option.some(add_rep_action_hom(r, g))
}

/// The bundled additive action homomorphism evaluates by the representation action.
theorem add_rep_action_hom_apply[G: Group, M: AddCommGroup](
    r: AddRepresentation[G, M], g: G, x: M
) {
    add_rep_action_hom(r, g).hom(x) = r.act(g, x)
} by {
    add_rep_action_hom_hom(r, g)
    add_rep_action_map_apply(r, g, x)
}

/// Group representation extensionality.
theorem add_rep_ext[G: Group, M: AddCommGroup](
    r: AddRepresentation[G, M],
    s: AddRepresentation[G, M]
) {
    (forall(g: G, x: M) { r.act(g, x) = s.act(g, x) }) implies r = s
} by {
    if forall(g: G, x: M) { r.act(g, x) = s.act(g, x) } {
        forall(g: G) {
            forall(x: M) {
                r.act(g, x) = s.act(g, x)
            }
            r.act(g) = s.act(g)
        }
        r.act = s.act
    }
}

/// The trivial action sends every pair to the second component.
let trivial_add_action[G: Group, M: AddCommGroup]: (G, M) -> M = function(g: G, x: M) {
    x
}

/// The trivial action is a multiplicative action.
theorem trivial_add_action_is_mul_action[G: Group, M: AddCommGroup] {
    is_mul_action(trivial_add_action[G, M])
} by {
    forall(g: G, h: G, x: M) {
        trivial_add_action[G, M](g, trivial_add_action[G, M](h, x)) = x
    }
    action_mul_constraint(trivial_add_action[G, M])
    forall(x: M) {
        trivial_add_action[G, M](G.1, x) = x
    }
    action_identity_constraint(trivial_add_action[G, M])
}

/// Each element acts by the identity additive homomorphism.
theorem trivial_add_action_is_add_hom[G: Group, M: AddCommGroup] {
    action_is_add_hom(trivial_add_action[G, M])
} by {
    forall(g: G) {
        forall(a: M, b: M) {
            trivial_add_action[G, M](g, b) = b
        }
        is_add_group_hom(function(z: M) { trivial_add_action[G, M](g, z) })
    }
}

/// The trivial action is an additive representation.
theorem trivial_add_action_is_add_representation[G: Group, M: AddCommGroup] {
    is_add_representation(trivial_add_action[G, M])
} by {
    trivial_add_action_is_mul_action[G, M]
    trivial_add_action_is_add_hom[G, M]
}

/// The trivial representation: every group element acts as the identity.
let trivial_add_representation[G: Group, M: AddCommGroup]: AddRepresentation[G, M] satisfy {
    AddRepresentation[G, M].new(trivial_add_action[G, M]) = Option.some(trivial_add_representation)
}

/// The trivial representation acts as the identity.
theorem trivial_add_representation_act[G: Group, M: AddCommGroup](g: G, x: M) {
    trivial_add_representation[G, M].act(g, x) = x
} by {
    trivial_add_representation[G, M].act = trivial_add_action[G, M]
    trivial_add_action[G, M](g, x) = x
}

/// True if a subset of the module is preserved by every group element of the representation.
define is_invariant_subset[G: Group, M: AddCommGroup](r: AddRepresentation[G, M], s: Set[M]) -> Bool {
    forall(g: G, x: M) {
        s.contains(x) implies s.contains(r.act(g, x))
    }
}

/// The whole module is invariant under every representation.
theorem universal_set_is_invariant_subset[G: Group, M: AddCommGroup](r: AddRepresentation[G, M]) {
    is_invariant_subset(r, Set[M].universal_set)
} by {
    forall(g: G, x: M) {
        if Set[M].universal_set.contains(x) {
            Set[M].universal_set.contains(r.act(g, x))
        }
    }
}


/// True if a group element fixes every element of the module.
define add_rep_acts_trivially[G: Group, M: AddCommGroup](r: AddRepresentation[G, M], g: G) -> Bool {
    forall(x: M) {
        r.act(g, x) = x
    }
}

/// The identity element acts trivially.
theorem add_rep_one_acts_trivially[G: Group, M: AddCommGroup](r: AddRepresentation[G, M]) {
    add_rep_acts_trivially(r, G.1)
} by {
    forall(x: M) {
        add_rep_one(r, x)
    }
}

/// True if no nontrivial group element acts trivially: the only element fixing every point is the identity.
define is_faithful_add_representation[G: Group, M: AddCommGroup](r: AddRepresentation[G, M]) -> Bool {
    forall(g: G) {
        add_rep_acts_trivially(r, g) implies g = G.1
    }
}

/// True if the representation commutes with the module's scalar multiplication:
/// every group element acts by an R-linear map on the underlying R-module.
define is_module_linear_add_representation[G: Group, R: Ring, M: AddCommGroup](
    rep: AddRepresentation[G, M],
    m: Module[R, M]
) -> Bool {
    forall(g: G, c: R, x: M) {
        rep.act(g, m.smul(c, x)) = m.smul(c, rep.act(g, x))
    }
}

/// An R-linear representation commutes the action with scalar multiplication.
theorem module_linear_add_rep_smul[G: Group, R: Ring, M: AddCommGroup](
    rep: AddRepresentation[G, M],
    m: Module[R, M],
    g: G,
    c: R,
    x: M
) {
    is_module_linear_add_representation(rep, m) implies
        rep.act(g, m.smul(c, x)) = m.smul(c, rep.act(g, x))
} by {
    if is_module_linear_add_representation(rep, m) {
        is_module_linear_add_representation(rep, m) = forall(h: G, d: R, y: M) {
            rep.act(h, m.smul(d, y)) = m.smul(d, rep.act(h, y))
        }
    }
}

/// An R-linear representation distributes over a sum scaled by a single coefficient.
theorem module_linear_add_rep_smul_add[G: Group, R: Ring, M: AddCommGroup](
    rep: AddRepresentation[G, M],
    m: Module[R, M],
    g: G,
    c: R,
    x: M,
    y: M
) {
    is_module_linear_add_representation(rep, m) implies
        rep.act(g, m.smul(c, x + y)) = m.smul(c, rep.act(g, x)) + m.smul(c, rep.act(g, y))
} by {
    if is_module_linear_add_representation(rep, m) {
        module_linear_add_rep_smul(rep, m, g, c, x + y)
        add_rep_act_add(rep, g, x, y)
        module_smul_add_right(m, c, rep.act(g, x), rep.act(g, y))
        rep.act(g, m.smul(c, x + y)) = m.smul(c, rep.act(g, x)) + m.smul(c, rep.act(g, y))
    }
}

/// An R-linear representation distributes over a linear combination c1 * x + c2 * y.
theorem module_linear_add_rep_lin_comb[G: Group, R: Ring, M: AddCommGroup](
    rep: AddRepresentation[G, M],
    m: Module[R, M],
    g: G,
    c1: R,
    c2: R,
    x: M,
    y: M
) {
    is_module_linear_add_representation(rep, m) implies
        rep.act(g, m.smul(c1, x) + m.smul(c2, y))
            = m.smul(c1, rep.act(g, x)) + m.smul(c2, rep.act(g, y))
} by {
    if is_module_linear_add_representation(rep, m) {
        add_rep_act_add(rep, g, m.smul(c1, x), m.smul(c2, y))
        module_linear_add_rep_smul(rep, m, g, c1, x)
        module_linear_add_rep_smul(rep, m, g, c2, y)
        rep.act(g, m.smul(c1, x) + m.smul(c2, y)) = m.smul(c1, rep.act(g, x)) + m.smul(c2, rep.act(g, y))
    }
}

/// An R-linear representation commutes with negation of the scalar.
theorem module_linear_add_rep_smul_neg[G: Group, R: Ring, M: AddCommGroup](
    rep: AddRepresentation[G, M],
    m: Module[R, M],
    g: G,
    c: R,
    x: M
) {
    is_module_linear_add_representation(rep, m) implies
        rep.act(g, m.smul(-c, x)) = m.smul(-c, rep.act(g, x))
} by {
    module_linear_add_rep_smul(rep, m, g, -c, x)
}

/// An R-linear representation sends a scaled zero to zero.
theorem module_linear_add_rep_smul_zero[G: Group, R: Ring, M: AddCommGroup](
    rep: AddRepresentation[G, M],
    m: Module[R, M],
    g: G,
    c: R
) {
    is_module_linear_add_representation(rep, m) implies
        rep.act(g, m.smul(c, M.0)) = m.smul(c, M.0)
} by {
    if is_module_linear_add_representation(rep, m) {
        module_linear_add_rep_smul(rep, m, g, c, M.0)
        add_rep_act_zero(rep, g)
        rep.act(g, M.0) = M.0
    }
}

/// A packaged R-linear representation: an additive representation paired with
/// a compatible R-module structure on the underlying additive group.
structure RingModuleRepresentation[G: Group, R: Ring, M: AddCommGroup] {
    /// The underlying additive representation.
    rep: AddRepresentation[G, M]
    /// The compatible R-module structure.
    module: Module[R, M]
} constraint {
    is_module_linear_add_representation(rep, module)
}

/// The action of a packaged R-linear representation commutes with scalar multiplication.
theorem ring_module_rep_smul[G: Group, R: Ring, M: AddCommGroup](
    p: RingModuleRepresentation[G, R, M],
    g: G,
    c: R,
    x: M
) {
    p.rep.act(g, p.module.smul(c, x)) = p.module.smul(c, p.rep.act(g, x))
} by {
    module_linear_add_rep_smul(p.rep, p.module, g, c, x)
}

/// The action of a packaged R-linear representation distributes over a linear combination.
theorem ring_module_rep_lin_comb[G: Group, R: Ring, M: AddCommGroup](
    p: RingModuleRepresentation[G, R, M],
    g: G,
    c1: R,
    c2: R,
    x: M,
    y: M
) {
    p.rep.act(g, p.module.smul(c1, x) + p.module.smul(c2, y))
        = p.module.smul(c1, p.rep.act(g, x)) + p.module.smul(c2, p.rep.act(g, y))
} by {
    module_linear_add_rep_lin_comb(p.rep, p.module, g, c1, c2, x, y)
}

/// The trivial representation is R-linear with respect to any module structure.
theorem trivial_add_representation_is_module_linear[G: Group, R: Ring, M: AddCommGroup](
    m: Module[R, M]
) {
    is_module_linear_add_representation(trivial_add_representation[G, M], m)
} by {
    forall(g: G, c: R, x: M) {
        trivial_add_representation_act(g, m.smul(c, x))
        trivial_add_representation_act(g, x)
        m.smul(c, trivial_add_representation[G, M].act(g, x)) = m.smul(c, x)
    }
}
