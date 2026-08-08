from comm_ring import CommRing
from algebra.ring.ideal import is_ideal, is_prime_ideal, is_prime_ideal_from_constraints,
    is_prime_ideal_mem, ideal_proper_constraint, ideal_prime_constraint,
    zero_ideal, zero_ideal_is_ideal
from nat import Nat, alt_induction

/// An integral domain is a commutative ring with distinct zero and one
/// in which the product of two nonzero elements is again nonzero.
typeclass D: IntegralDomain extends CommRing {
    /// Zero and one are distinct elements of the integral domain.
    zero_ne_one {
        D.0 != D.1
    }

    /// The product of two elements of an integral domain is zero only if at least one factor is zero.
    no_zero_divisors(a: D, b: D) {
        a * b = D.0 implies a = D.0 or b = D.0
    }
}

/// In an integral domain, multiplication by a nonzero element is left-cancellative.
theorem integral_domain_mul_left_cancel[D: IntegralDomain](a: D, b: D, c: D) {
    a != D.0 and a * b = a * c implies b = c
} by {
    if a != D.0 and a * b = a * c {
        a * b - a * c = D.0
        a * (b - c) = a * b - a * c
        a * (b - c) = D.0
        D.no_zero_divisors(a, b - c)
        a = D.0 or b - c = D.0
        b - c = D.0
        b - c + c = D.0 + c
        b - c + c = b
        b = c
    }
}

/// In an integral domain, multiplication by a nonzero element is right-cancellative.
theorem integral_domain_mul_right_cancel[D: IntegralDomain](a: D, b: D, c: D) {
    c != D.0 and a * c = b * c implies a = b
} by {
    if c != D.0 and a * c = b * c {
        a * c - b * c = D.0
        (a - b) * c = a * c - b * c
        (a - b) * c = D.0
        D.no_zero_divisors(a - b, c)
        a - b = D.0 or c = D.0
        a - b = D.0
        a - b + b = D.0 + b
        a - b + b = a
        a = b
    }
}

/// In an integral domain, a zero product has a zero factor.
theorem integral_domain_mul_eq_zero[D: IntegralDomain](a: D, b: D) {
    a * b = D.0 implies a = D.0 or b = D.0
} by {
    if a * b = D.0 {
        D.no_zero_divisors(a, b)
        a = D.0 or b = D.0
    }
}

/// If either factor is zero, then the product is zero.
theorem integral_domain_mul_eq_zero_of_factor[D: IntegralDomain](a: D, b: D) {
    a = D.0 or b = D.0 implies a * b = D.0
} by {
    if a = D.0 or b = D.0 {
        if a = D.0 {
            a * b = D.0
        } else {
            a * b = D.0
        }
    }
}

/// In an integral domain, the product of two nonzero elements is nonzero.
theorem integral_domain_mul_nonzero[D: IntegralDomain](a: D, b: D) {
    a != D.0 and b != D.0 implies a * b != D.0
} by {
    if a != D.0 and b != D.0 {
        if a * b = D.0 {
            D.no_zero_divisors(a, b)
            false
        }
    }
}

/// Nonzero products can be peeled on the left.
theorem integral_domain_left_nonzero_of_mul_nonzero[D: IntegralDomain](a: D, b: D) {
    a * b != D.0 implies a != D.0
} by {
    if a * b != D.0 {
        if a = D.0 {
            D.0 * b = D.0
            false
        }
    }
}

/// Nonzero products can be peeled on the right.
theorem integral_domain_right_nonzero_of_mul_nonzero[D: IntegralDomain](a: D, b: D) {
    a * b != D.0 implies b != D.0
} by {
    if a * b != D.0 {
        if b = D.0 {
            a * D.0 = D.0
            false
        }
    }
}

/// Squaring a nonzero element in an integral domain is nonzero.
theorem integral_domain_square_nonzero[D: IntegralDomain](a: D) {
    a != D.0 implies a * a != D.0
} by {
    if a != D.0 {
        integral_domain_mul_nonzero(a, a)
    }
}

/// Multiplying a nonzero element by a nonzero power gives a nonzero successor power.
theorem integral_domain_pow_suc_nonzero[D: IntegralDomain](a: D, k: Nat) {
    a != D.0 and a.pow(k) != D.0 implies a.pow(k.suc) != D.0
} by {
    if a != D.0 and a.pow(k) != D.0 {
        integral_domain_mul_nonzero(a, a.pow(k))
        a.pow(k.suc) != D.0
    }
}

/// In an integral domain, every power of a nonzero element is nonzero.
theorem integral_domain_pow_nonzero[D: IntegralDomain](a: D, n: Nat) {
    a != D.0 implies a.pow(n) != D.0
} by {
    if a != D.0 {
        define p(k: Nat) -> Bool {
            a.pow(k) != D.0
        }
        a.pow(Nat.0) = D.1
        D.zero_ne_one
        D.1 != D.0
        p(Nat.0)
        forall(k: Nat) {
            if p(k) {
                integral_domain_pow_suc_nonzero(a, k)
                p(k.suc)
            }
        }
        alt_induction(p)
        forall(k: Nat) {
            p(k)
        }
        a.pow(n) != D.0
    }
}

/// In an integral domain, the zero ideal is a prime ideal.
theorem zero_ideal_is_prime_of_integral_domain[D: IntegralDomain] {
    is_prime_ideal(zero_ideal[D])
} by {
    zero_ideal_is_ideal[D]

    D.zero_ne_one
    ideal_proper_constraint(zero_ideal[D])

    forall(a: D, b: D) {
        if zero_ideal[D](a * b) {
            D.no_zero_divisors(a, b)
            zero_ideal[D](a) or zero_ideal[D](b)
        }
    }
    ideal_prime_constraint(zero_ideal[D])

    is_prime_ideal_from_constraints(zero_ideal[D])
}

/// Conversely, if the zero ideal of a commutative ring is prime, then the ring has no zero divisors.
theorem no_zero_divisors_of_zero_ideal_prime[R: CommRing](a: R, b: R) {
    is_prime_ideal(zero_ideal[R]) and a * b = R.0 implies a = R.0 or b = R.0
} by {
    if is_prime_ideal(zero_ideal[R]) and a * b = R.0 {
        is_prime_ideal_mem(zero_ideal[R], a, b)
        zero_ideal[R](a) or zero_ideal[R](b)
    }
}
