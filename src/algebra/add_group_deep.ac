// Additive-group deepening: the elementary laws of negation and subtraction
// in a general (not necessarily commutative) additive group, stated at the
// `AddGroup` level rather than the ring level.

from algebra.add_group import AddGroup, inverse_inverse, inverse_left, inverse_add, left_cancel
from algebra.add_comm_group import AddCommGroup
from algebra.add_comm_monoid import AddCommMonoid

/// The additive identity is its own additive inverse.
theorem add_group_neg_zero[A: AddGroup] {
    -A.0 = A.0
} by {
    A.0 + -A.0 = A.0
    A.0 + A.0 = A.0
    -A.0 = A.0
}

/// Subtracting an element from itself gives zero.
theorem add_group_sub_self[A: AddGroup](a: A) {
    a - a = A.0
} by {
    a - a = a + -a
    a + -a = A.0
}

/// Subtracting zero leaves an element unchanged.
theorem add_group_sub_zero[A: AddGroup](a: A) {
    a - A.0 = a
} by {
    a - A.0 = a + -A.0
    add_group_neg_zero[A]
    -A.0 = A.0
    a + A.0 = a
}

/// Subtracting an element from zero gives its additive inverse.
theorem add_group_zero_sub[A: AddGroup](a: A) {
    A.0 - a = -a
} by {
    A.0 - a = A.0 + -a
    A.0 + -a = -a
}

/// A difference is zero exactly when the two elements are equal.
theorem add_group_sub_eq_zero_iff[A: AddGroup](a: A, b: A) {
    (a - b = A.0) = (a = b)
} by {
    if a - b = A.0 {
        (a - b) + b = A.0 + b
        (a - b) + b = a + (-b + b)
        inverse_left(b)
        -b + b = A.0
        a + (-b + b) = a + A.0
        a + A.0 = a
        (a - b) + b = a
        A.0 + b = b
        a = b
    }
    if a = b {
        a - b = a - a
        add_group_sub_self(a)
        a - a = A.0
        a - b = A.0
    }
    (a - b = A.0) = (a = b)
}

/// The additive inverse of a difference is the reversed difference.
theorem add_group_neg_sub[A: AddGroup](a: A, b: A) {
    -(a - b) = b - a
} by {
    a - b = a + -b
    -(a - b) = -(a + -b)
    inverse_add(a, -b)
    -(a + -b) = --b + -a
    inverse_inverse(b)
    --b = b
    --b + -a = b + -a
    b - a = b + -a
    -(a - b) = b - a
}

/// Subtracting a negated element is adding the element.
theorem add_group_sub_neg[A: AddGroup](a: A, b: A) {
    a - -b = a + b
} by {
    a - -b = a + --b
    inverse_inverse(b)
    --b = b
    a + --b = a + b
}

/// A sum is zero exactly when the second summand is the negation of the first.
theorem add_group_add_eq_zero_iff_eq_neg[A: AddGroup](a: A, b: A) {
    (a + b = A.0) = (b = -a)
} by {
    if a + b = A.0 {
        inverse_inverse(a)
        left_cancel(a, b, -a)
        a + -a = A.0
        b = -a
    }
    if b = -a {
        a + b = a + -a
        a + -a = A.0
        a + b = A.0
    }
    (a + b = A.0) = (b = -a)
}

/// An element is zero exactly when its additive inverse is zero.
theorem add_group_neg_eq_zero_iff[A: AddGroup](a: A) {
    (-a = A.0) = (a = A.0)
} by {
    if -a = A.0 {
        inverse_inverse(a)
        --a = a
        -a = A.0
        a = A.0
    }
    if a = A.0 {
        -a = -A.0
        add_group_neg_zero[A]
        -A.0 = A.0
        -a = A.0
    }
    (-a = A.0) = (a = A.0)
}

/// Adding back the subtracted element to a difference recovers the first element.
theorem add_group_sub_add_cancel[A: AddGroup](a: A, b: A) {
    (a - b) + b = a
} by {
    a - b = a + -b
    (a - b) + b = (a + -b) + b
    (a + -b) + b = a + (-b + b)
    inverse_left(b)
    -b + b = A.0
    a + (-b + b) = a + A.0
    a + A.0 = a
    (a - b) + b = a
}

/// Subtracting a summand from a sum recovers the other summand.
theorem add_group_add_sub_cancel[A: AddGroup](a: A, b: A) {
    a + b - b = a
} by {
    a + b - b = (a + b) + -b
    (a + b) + -b = a + (b + -b)
    b + -b = A.0
    a + (b + -b) = a + A.0
    a + A.0 = a
    a + b - b = a
}

/// A double subtraction in a commutative additive group: a - (b - c) = a - b + c.
theorem add_comm_group_sub_sub[A: AddCommGroup](a: A, b: A, c: A) {
    a - (b - c) = a - b + c
} by {
    b - c = b + -c
    a - (b - c) = a - (b + -c)
    a - (b + -c) = a + -(b + -c)
    inverse_add(b, -c)
    -(b + -c) = --c + -b
    inverse_inverse(c)
    --c = c
    --c + -b = c + -b
    a + (c + -b) = a + c + -b
    a - b + c = a + -b + c
    a + c + -b = a + -b + c
    a - (b - c) = a - b + c
}

/// An element equals its negation exactly when it doubles to zero.
theorem add_group_neg_eq_self_iff_add_self_zero[A: AddGroup](a: A) {
    (-a = a) = (a + a = A.0)
} by {
    if -a = a {
        a + a = a + -a
        a + -a = A.0
        a + a = A.0
    }
    if a + a = A.0 {
        add_group_add_eq_zero_iff_eq_neg(a, a)
        (a + a = A.0) = (a = -a)
        a = -a
        -a = a
    }
    (-a = a) = (a + a = A.0)
}
