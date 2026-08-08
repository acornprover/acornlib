from algebra.group import Group, inverse_left
from algebra.subgroup import Subgroup, subgroup_constraint, identity_constraint, closure_constraint,
    inverse_constraint
from data.basic.set import Set
from data.basic.functions import identity_fn, compose, function_extensionality, is_injective_fn,
    is_surjective_fn, is_bijection_fn, injective_fn_eq, Bijection, bijection_new_map
from data.basic.relation_basic import is_reflexive, is_symmetric

/// True if the identity element acts trivially.
define action_identity_constraint[G: Group, X](act: (G, X) -> X) -> Bool {
    forall(x: X) {
        act(G.1, x) = x
    }
}

/// True if multiplication in the group is composition of actions.
define action_mul_constraint[G: Group, X](act: (G, X) -> X) -> Bool {
    forall(g: G, h: G, x: X) {
        act(g * h, x) = act(g, act(h, x))
    }
}

/// True if a binary operation is a left action of a group on a type.
define is_mul_action[G: Group, X](act: (G, X) -> X) -> Bool {
    action_mul_constraint(act) and action_identity_constraint(act)
}

/// A left action of a group on a type.
structure MulAction[G: Group, X] {
    /// The action map.
    act: (G, X) -> X
} constraint {
    is_mul_action(act)
}

/// The identity element acts trivially.
theorem mul_action_one[G: Group, X](a: MulAction[G, X], x: X) {
    a.act(G.1, x) = x
} by {
    action_identity_constraint(a.act)
}

/// Multiplication in the group is composition of actions.
theorem mul_action_mul[G: Group, X](a: MulAction[G, X], g: G, h: G, x: X) {
    a.act(g * h, x) = a.act(g, a.act(h, x))
} by {
    is_mul_action(a.act)
    action_mul_constraint(a.act)
    action_mul_constraint(a.act) = forall(k: G, l: G, y: X) {
        a.act(k * l, y) = a.act(k, a.act(l, y))
    }
}

/// Acting by the inverse on the left cancels an action.
theorem mul_action_inv_apply_apply[G: Group, X](a: MulAction[G, X], g: G, x: X) {
    a.act(g.inverse, a.act(g, x)) = x
} by {
    inverse_left(g)
    mul_action_mul(a, g.inverse, g, x)
    mul_action_one(a, x)
}

/// Acting by the inverse on the right cancels an action.
theorem mul_action_apply_inv_apply[G: Group, X](a: MulAction[G, X], g: G, x: X) {
    a.act(g, a.act(g.inverse, x)) = x
} by {
    mul_action_mul(a, g, g.inverse, x)
    mul_action_one(a, x)
}

/// The permutation of the underlying type induced by a group element.
define action_map[G: Group, X](a: MulAction[G, X], g: G, x: X) -> X {
    a.act(g, x)
}

/// The induced map of a group element evaluates by the action.
theorem action_map_apply[G: Group, X](a: MulAction[G, X], g: G, x: X) {
    action_map(a, g, x) = a.act(g, x)
}

/// The inverse group element gives a left inverse for the induced action map.
theorem action_map_inverse_apply_apply[G: Group, X](a: MulAction[G, X], g: G, x: X) {
    action_map(a, g.inverse, action_map(a, g, x)) = x
} by {
    action_map_apply(a, g, x)
    action_map_apply(a, g.inverse, action_map(a, g, x))
    mul_action_inv_apply_apply(a, g, x)
}

/// The inverse group element gives a right inverse for the induced action map.
theorem action_map_apply_inverse_apply[G: Group, X](a: MulAction[G, X], g: G, x: X) {
    action_map(a, g, action_map(a, g.inverse, x)) = x
} by {
    action_map_apply(a, g.inverse, x)
    action_map_apply(a, g, action_map(a, g.inverse, x))
    mul_action_apply_inv_apply(a, g, x)
}

/// The induced map of a group element is injective.
theorem action_map_is_injective[G: Group, X](a: MulAction[G, X], g: G) {
    is_injective_fn(action_map(a, g))
} by {
    forall(x: X, y: X) {
        if action_map(a, g, x) = action_map(a, g, y) {
            action_map_inverse_apply_apply(a, g, x)
            action_map_inverse_apply_apply(a, g, y)
            x = y
        }
    }
}

/// The induced map of a group element is surjective.
theorem action_map_is_surjective[G: Group, X](a: MulAction[G, X], g: G) {
    is_surjective_fn(action_map(a, g))
} by {
    forall(y: X) {
        let x = action_map(a, g.inverse, y)
        action_map_apply_inverse_apply(a, g, y)
        exists(z: X) {
            z = x and action_map(a, g, z) = y
        }
    }
}

/// The induced map of a group element is a bijection.
theorem action_map_is_bijection[G: Group, X](a: MulAction[G, X], g: G) {
    is_bijection_fn(action_map(a, g))
} by {
    action_map_is_injective(a, g)
    action_map_is_surjective(a, g)
}

/// The induced map of a group element as a bundled bijection.
let action_bijection[G: Group, X](a: MulAction[G, X], g: G) -> result: Bijection[X, X] satisfy {
    result.map = action_map(a, g)
} by {
    action_map_is_bijection(a, g)
    let e: Bijection[X, X] satisfy {
        Bijection[X, X].new(action_map(a, g)) = Option.some(e)
    }
    bijection_new_map(action_map(a, g), e)
}

/// The bundled action bijection has the induced action map as its underlying map.
theorem action_bijection_map[G: Group, X](a: MulAction[G, X], g: G) {
    action_bijection(a, g).map = action_map(a, g)
}

/// The bundled action bijection evaluates by the action.
theorem action_bijection_apply[G: Group, X](a: MulAction[G, X], g: G, x: X) {
    action_bijection(a, g).map(x) = a.act(g, x)
} by {
    action_bijection_map(a, g)
    action_map_apply(a, g, x)
}

/// The identity element induces the identity map.
theorem action_map_one[G: Group, X](a: MulAction[G, X]) {
    action_map(a, G.1) = identity_fn[X]
} by {
    forall(x: X) {
        mul_action_one(a, x)
        action_map(a, G.1, x) = identity_fn[X](x)
    }
    function_extensionality(action_map(a, G.1), identity_fn[X])
}

/// The identity action bijection has the identity map.
theorem action_bijection_one_map[G: Group, X](a: MulAction[G, X]) {
    action_bijection(a, G.1).map = identity_fn[X]
} by {
    action_bijection_map(a, G.1)
    action_map_one(a)
}

/// Multiplication of group elements induces composition of action maps.
theorem action_map_mul[G: Group, X](a: MulAction[G, X], g: G, h: G) {
    action_map(a, g * h) = compose(action_map(a, g), action_map(a, h))
} by {
    forall(x: X) {
        mul_action_mul(a, g, h, x)
        action_map(a, g * h, x) = compose(action_map(a, g), action_map(a, h), x)
    }
    function_extensionality(action_map(a, g * h), compose(action_map(a, g), action_map(a, h)))
}

/// Multiplication of group elements induces composition of bundled action-bijection maps.
theorem action_bijection_mul_map[G: Group, X](a: MulAction[G, X], g: G, h: G) {
    action_bijection(a, g * h).map =
        compose(action_bijection(a, g).map, action_bijection(a, h).map)
} by {
    action_bijection_map(a, g * h)
    action_bijection_map(a, g)
    action_bijection_map(a, h)
    action_map_mul(a, g, h)
}

/// The inverse action map is a left inverse under composition.
theorem action_map_inverse_compose_left[G: Group, X](a: MulAction[G, X], g: G) {
    compose(action_map(a, g.inverse), action_map(a, g)) = identity_fn[X]
} by {
    forall(x: X) {
        action_map_inverse_apply_apply(a, g, x)
        compose(action_map(a, g.inverse), action_map(a, g), x) = identity_fn[X](x)
    }
    function_extensionality(compose(action_map(a, g.inverse), action_map(a, g)), identity_fn[X])
}

/// The inverse action map is a right inverse under composition.
theorem action_map_inverse_compose_right[G: Group, X](a: MulAction[G, X], g: G) {
    compose(action_map(a, g), action_map(a, g.inverse)) = identity_fn[X]
} by {
    forall(x: X) {
        action_map_apply_inverse_apply(a, g, x)
        compose(action_map(a, g), action_map(a, g.inverse), x) = identity_fn[X](x)
    }
    function_extensionality(compose(action_map(a, g), action_map(a, g.inverse)), identity_fn[X])
}

/// True if a map commutes with two actions of the same group.
define is_equivariant_map[G: Group, X, Y](
    a: MulAction[G, X],
    b: MulAction[G, Y],
    f: X -> Y
) -> Bool {
    forall(g: G, x: X) {
        f(a.act(g, x)) = b.act(g, f(x))
    }
}

/// An equivariant map between two actions of the same group.
structure ActionHom[G: Group, X, Y] {
    /// The source action.
    src: MulAction[G, X]

    /// The target action.
    dst: MulAction[G, Y]

    /// The underlying map.
    map: X -> Y
} constraint {
    is_equivariant_map(src, dst, map)
}

/// An equivariant map commutes with the action of every group element.
theorem equivariant_map_apply[G: Group, X, Y](
    a: MulAction[G, X],
    b: MulAction[G, Y],
    f: X -> Y,
    g: G,
    x: X
) {
    is_equivariant_map(a, b, f) implies f(a.act(g, x)) = b.act(g, f(x))
} by {
    if is_equivariant_map(a, b, f) {
        is_equivariant_map(a, b, f) = forall(h: G, y: X) {
            f(a.act(h, y)) = b.act(h, f(y))
        }
    }
}

/// The underlying map of an action homomorphism is equivariant.
theorem action_hom_is_equivariant[G: Group, X, Y](f: ActionHom[G, X, Y]) {
    is_equivariant_map(f.src, f.dst, f.map)
}

/// An action homomorphism commutes with the action of every group element.
theorem action_hom_apply[G: Group, X, Y](f: ActionHom[G, X, Y], g: G, x: X) {
    f.map(f.src.act(g, x)) = f.dst.act(g, f.map(x))
} by {
    action_hom_is_equivariant(f)
    equivariant_map_apply(f.src, f.dst, f.map, g, x)
}

/// Action homomorphism extensionality from equality of fields.
theorem action_hom_ext[G: Group, X, Y](f: ActionHom[G, X, Y], h: ActionHom[G, X, Y]) {
    f.src = h.src and f.dst = h.dst and f.map = h.map implies f = h
}

/// The extensionality attribute for action homomorphisms.
attributes ActionHom[G: Group, X, Y] {
    /// Action homomorphism extensionality from equality of fields.
    let ext = action_hom_ext[G, X, Y]
}

/// Equal action homomorphisms have equal underlying maps.
theorem action_hom_eq_map[G: Group, X, Y](f: ActionHom[G, X, Y], h: ActionHom[G, X, Y]) {
    f = h implies f.map = h.map
}

/// Equal action homomorphisms have equal values at every point.
theorem action_hom_eq_apply[G: Group, X, Y](f: ActionHom[G, X, Y], h: ActionHom[G, X, Y], x: X) {
    f = h implies f.map(x) = h.map(x)
} by {
    if f = h {
        f.map(x) = h.map(x)
    }
}

/// The identity map is equivariant for any action.
theorem identity_fn_is_equivariant[G: Group, X](a: MulAction[G, X]) {
    is_equivariant_map(a, a, identity_fn[X])
} by {
    forall(g: G, x: X) {
        identity_fn[X](a.act(g, x)) = a.act(g, identity_fn[X](x))
    }
}

/// The identity action homomorphism.
let identity_action_hom[G: Group, X](a: MulAction[G, X]) -> result: ActionHom[G, X, X] satisfy {
    ActionHom[G, X, X].new(a, a, identity_fn[X]) = Option.some(result)
} by {
    identity_fn_is_equivariant(a)
}

/// The identity action homomorphism has the identity map.
theorem identity_action_hom_map[G: Group, X](a: MulAction[G, X]) {
    identity_action_hom(a).map = identity_fn[X]
} by {
    identity_action_hom(a).map = identity_fn[X]
}

/// The identity action homomorphism fixes every point.
theorem identity_action_hom_apply[G: Group, X](a: MulAction[G, X], x: X) {
    identity_action_hom(a).map(x) = x
} by {
    identity_action_hom_map(a)
}

/// The composite of equivariant maps is equivariant.
theorem compose_is_equivariant_map[G: Group, X, Y, Z](
    a: MulAction[G, X],
    b: MulAction[G, Y],
    c: MulAction[G, Z],
    f: Y -> Z,
    h: X -> Y
) {
    is_equivariant_map(b, c, f) and is_equivariant_map(a, b, h) implies
        is_equivariant_map(a, c, compose(f, h))
} by {
    if is_equivariant_map(b, c, f) and is_equivariant_map(a, b, h) {
        forall(g: G, x: X) {
            equivariant_map_apply(a, b, h, g, x)
            equivariant_map_apply(b, c, f, g, h(x))
            compose(f, h, a.act(g, x)) = c.act(g, compose(f, h, x))
        }
    }
}

/// Composable action homomorphisms have an equivariant composite map.
theorem action_hom_compose_maps_is_equivariant[G: Group, X, Y, Z](
    f: ActionHom[G, Y, Z],
    h: ActionHom[G, X, Y]
) {
    h.dst = f.src implies is_equivariant_map(h.src, f.dst, compose(f.map, h.map))
} by {
    if h.dst = f.src {
        action_hom_is_equivariant(f)
        action_hom_is_equivariant(h)
        is_equivariant_map(h.src, f.src, h.map)
        compose_is_equivariant_map(h.src, f.src, f.dst, f.map, h.map)
    }
}

/// The composite of two action homomorphism maps evaluates by composing their values.
theorem action_hom_compose_maps_apply[G: Group, X, Y, Z](
    f: ActionHom[G, Y, Z],
    h: ActionHom[G, X, Y],
    x: X
) {
    compose(f.map, h.map, x) = f.map(h.map(x))
}

/// True if one point belongs to the orbit of another point.
define orbit_contains[G: Group, X](a: MulAction[G, X], x: X, y: X) -> Bool {
    exists(g: G) {
        y = a.act(g, x)
    }
}

/// True if two points lie in the same orbit.
define orbit_relation[G: Group, X](a: MulAction[G, X], x: X, y: X) -> Bool {
    orbit_contains(a, x, y)
}

/// The orbit of a point under a group action.
define orbit[G: Group, X](a: MulAction[G, X], x: X) -> Set[X] {
    Set[X].new(orbit_contains(a, x))
}

/// A point belongs to its own orbit.
theorem orbit_contains_self[G: Group, X](a: MulAction[G, X], x: X) {
    orbit(a, x).contains(x)
} by {
    mul_action_one(a, x)
}

/// Acting on a point gives a member of its orbit.
theorem orbit_contains_action[G: Group, X](a: MulAction[G, X], x: X, g: G) {
    orbit(a, x).contains(a.act(g, x))
} by {
    exists(h: G) {
        h = g and a.act(g, x) = a.act(h, x)
    }
}

/// Membership in an orbit has a group element witness.
theorem orbit_contains_witness[G: Group, X](a: MulAction[G, X], x: X, y: X) {
    orbit(a, x).contains(y) implies exists(g: G) {
        y = a.act(g, x)
    }
} by {
    if orbit(a, x).contains(y) {
        orbit(a, x).contains(y) = orbit_contains(a, x, y)
    }
}

/// An equivariant map sends orbit membership to orbit membership.
theorem equivariant_map_maps_orbit_contains[G: Group, X, Y](
    a: MulAction[G, X],
    b: MulAction[G, Y],
    f: X -> Y,
    x: X,
    y: X
) {
    is_equivariant_map(a, b, f) and orbit(a, x).contains(y) implies orbit(b, f(x)).contains(f(y))
} by {
    if is_equivariant_map(a, b, f) and orbit(a, x).contains(y) {
        orbit_contains_witness(a, x, y)
        let g: G satisfy {
            y = a.act(g, x)
        }
        equivariant_map_apply(a, b, f, g, x)
        orbit_contains_action(b, f(x), g)
        orbit(b, f(x)).contains(f(y))
    }
}

/// An action homomorphism sends orbit membership to orbit membership.
theorem action_hom_maps_orbit_contains[G: Group, X, Y](
    f: ActionHom[G, X, Y],
    x: X,
    y: X
) {
    orbit(f.src, x).contains(y) implies orbit(f.dst, f.map(x)).contains(f.map(y))
} by {
    if orbit(f.src, x).contains(y) {
        action_hom_is_equivariant(f)
        equivariant_map_maps_orbit_contains(f.src, f.dst, f.map, x, y)
    }
}

/// An injective equivariant map reflects orbit membership.
theorem equivariant_map_reflects_orbit_contains_of_injective[G: Group, X, Y](
    a: MulAction[G, X],
    b: MulAction[G, Y],
    f: X -> Y,
    x: X,
    y: X
) {
    is_injective_fn(f) and is_equivariant_map(a, b, f) and
        orbit(b, f(x)).contains(f(y)) implies orbit(a, x).contains(y)
} by {
    if is_injective_fn(f) and is_equivariant_map(a, b, f) and
        orbit(b, f(x)).contains(f(y)) {
        orbit_contains_witness(b, f(x), f(y))
        let g: G satisfy {
            f(y) = b.act(g, f(x))
        }
        equivariant_map_apply(a, b, f, g, x)
        injective_fn_eq(f, y, a.act(g, x))
        y = a.act(g, x)
        orbit_contains_action(a, x, g)
        orbit(a, x).contains(y)
    }
}

/// An injective action homomorphism reflects orbit membership.
theorem action_hom_reflects_orbit_contains_of_injective[G: Group, X, Y](
    f: ActionHom[G, X, Y],
    x: X,
    y: X
) {
    is_injective_fn(f.map) and orbit(f.dst, f.map(x)).contains(f.map(y)) implies
        orbit(f.src, x).contains(y)
} by {
    if is_injective_fn(f.map) and orbit(f.dst, f.map(x)).contains(f.map(y)) {
        action_hom_is_equivariant(f)
        equivariant_map_reflects_orbit_contains_of_injective(f.src, f.dst, f.map, x, y)
    }
}

/// Orbit membership is symmetric.
theorem orbit_contains_symmetric[G: Group, X](a: MulAction[G, X], x: X, y: X) {
    orbit(a, x).contains(y) implies orbit(a, y).contains(x)
} by {
    if orbit(a, x).contains(y) {
        orbit_contains_witness(a, x, y)
        let g: G satisfy {
            y = a.act(g, x)
        }
        mul_action_inv_apply_apply(a, g, x)
        exists(h: G) {
            h = g.inverse and x = a.act(h, y)
        }
    }
}

/// Acting on an orbit member stays in the same orbit.
theorem orbit_contains_action_of_contains[G: Group, X](a: MulAction[G, X], x: X, y: X, g: G) {
    orbit(a, x).contains(y) implies orbit(a, x).contains(a.act(g, y))
} by {
    if orbit(a, x).contains(y) {
        orbit_contains_witness(a, x, y)
        let h: G satisfy {
            y = a.act(h, x)
        }
        mul_action_mul(a, g, h, x)
        orbit_contains_action(a, x, g * h)
        orbit(a, x).contains(a.act(g, y))
    }
}

/// Orbit membership is transitive.
theorem orbit_contains_transitive[G: Group, X](a: MulAction[G, X], x: X, y: X, z: X) {
    orbit(a, x).contains(y) and orbit(a, y).contains(z) implies orbit(a, x).contains(z)
} by {
    if orbit(a, x).contains(y) and orbit(a, y).contains(z) {
        orbit_contains_witness(a, y, z)
        let h: G satisfy {
            z = a.act(h, y)
        }
        orbit_contains_action_of_contains(a, x, y, h)
        orbit(a, x).contains(z)
    }
}

/// Orbit relation membership is exactly orbit-set membership.
theorem orbit_relation_eq_contains[G: Group, X](a: MulAction[G, X], x: X, y: X) {
    orbit_relation(a, x, y) = orbit(a, x).contains(y)
} by {
}

/// The orbit relation is reflexive.
theorem orbit_relation_is_reflexive[G: Group, X](a: MulAction[G, X]) {
    is_reflexive(orbit_relation(a))
} by {
    forall(x: X) {
        orbit_contains_self(a, x)
        orbit_relation(a, x, x)
    }
}

/// The orbit relation is symmetric.
theorem orbit_relation_is_symmetric[G: Group, X](a: MulAction[G, X]) {
    is_symmetric(orbit_relation(a))
} by {
    forall(x: X, y: X) {
        if orbit_relation(a, x, y) {
            orbit_contains_symmetric(a, x, y)
            orbit(a, y).contains(x)
            orbit_relation(a, y, x)
        }
    }
}

/// Two points in the same orbit have the same orbit.
theorem orbit_eq_of_contains[G: Group, X](a: MulAction[G, X], x: X, y: X) {
    orbit(a, x).contains(y) implies orbit(a, x) = orbit(a, y)
} by {
    if orbit(a, x).contains(y) {
        forall(z: X) {
            if orbit(a, x).contains(z) {
                orbit_contains_symmetric(a, x, y)
                orbit_contains_transitive(a, y, x, z)
                orbit(a, y).contains(z)
            }
            if orbit(a, y).contains(z) {
                orbit_contains_transitive(a, x, y, z)
                orbit(a, x).contains(z)
            }
            orbit(a, x).contains(z) = orbit(a, y).contains(z)
        }
        orbit(a, x) = orbit(a, y)
    }
}

/// Equality of orbits is equivalent to membership of the second representative in the first orbit.
theorem orbit_eq_iff_contains[G: Group, X](a: MulAction[G, X], x: X, y: X) {
    orbit(a, x) = orbit(a, y) = orbit(a, x).contains(y)
} by {
    if orbit(a, x) = orbit(a, y) {
        orbit_contains_self(a, y)
        orbit(a, x).contains(y)
    }
    if orbit(a, x).contains(y) {
        orbit_eq_of_contains(a, x, y)
        orbit(a, x) = orbit(a, y)
    }
}


/// True if a group element fixes a point.
define stabilizer_contains[G: Group, X](a: MulAction[G, X], x: X, g: G) -> Bool {
    a.act(g, x) = x
}

/// The identity element fixes every point.
theorem stabilizer_identity_constraint[G: Group, X](a: MulAction[G, X], x: X) {
    identity_constraint(stabilizer_contains(a, x))
} by {
    mul_action_one(a, x)
}

/// The product of two elements fixing a point fixes that point.
theorem stabilizer_closure_constraint[G: Group, X](a: MulAction[G, X], x: X) {
    closure_constraint(stabilizer_contains(a, x))
} by {
    forall(g: G, h: G) {
        if stabilizer_contains(a, x, g) and stabilizer_contains(a, x, h) {
            a.act(g, x) = x
            mul_action_mul(a, g, h, x)
            a.act(g, a.act(h, x)) = a.act(g, x)
            stabilizer_contains(a, x, g * h)
        }
    }
}

/// The inverse of an element fixing a point fixes that point.
theorem stabilizer_inverse_constraint[G: Group, X](a: MulAction[G, X], x: X) {
    inverse_constraint(stabilizer_contains(a, x))
} by {
    forall(g: G) {
        if stabilizer_contains(a, x, g) {
            a.act(g, x) = x
            mul_action_inv_apply_apply(a, g, x)
            stabilizer_contains(a, x, g.inverse)
        }
    }
}

/// The elements fixing a point form a subgroup.
theorem stabilizer_subgroup_constraint[G: Group, X](a: MulAction[G, X], x: X) {
    subgroup_constraint(stabilizer_contains(a, x))
} by {
    stabilizer_identity_constraint(a, x)
    stabilizer_closure_constraint(a, x)
    stabilizer_inverse_constraint(a, x)
}

/// The stabilizer subgroup of a point.
let stabilizer[G: Group, X](a: MulAction[G, X], x: X) -> result: Subgroup[G] satisfy {
    Subgroup.new(stabilizer_contains(a, x)) = Option.some(result)
} by {
    stabilizer_subgroup_constraint(a, x)
}

/// Membership in the stabilizer is fixing the point.
theorem stabilizer_contains_eq[G: Group, X](a: MulAction[G, X], x: X, g: G) {
    stabilizer(a, x).contains(g) = (a.act(g, x) = x)
} by {
    stabilizer(a, x).contains(g) = stabilizer_contains(a, x, g)
}

/// A stabilizer member fixes the point.
theorem stabilizer_fixes[G: Group, X](a: MulAction[G, X], x: X, g: G) {
    stabilizer(a, x).contains(g) implies a.act(g, x) = x
} by {
    if stabilizer(a, x).contains(g) {
        stabilizer_contains_eq(a, x, g)
    }
}

/// A group element fixing the point belongs to its stabilizer.
theorem stabilizer_contains_of_fixes[G: Group, X](a: MulAction[G, X], x: X, g: G) {
    a.act(g, x) = x implies stabilizer(a, x).contains(g)
} by {
    if a.act(g, x) = x {
        stabilizer_contains_eq(a, x, g)
        stabilizer(a, x).contains(g)
    }
}

/// Two group elements act equally on a point exactly when their quotient lies in the stabilizer.
theorem orbit_action_eq_iff_stabilizer_left_coset_raw[G: Group, X](
    a: MulAction[G, X],
    x: X,
    g: G,
    k: G
) {
    (a.act(g, x) = a.act(k, x)) = stabilizer(a, x).contains(g.inverse * k)
} by {
    if a.act(g, x) = a.act(k, x) {
        mul_action_mul(a, g.inverse, k, x)
        mul_action_inv_apply_apply(a, g, x)
        stabilizer_contains_eq(a, x, g.inverse * k)
        stabilizer(a, x).contains(g.inverse * k)
    }
    if stabilizer(a, x).contains(g.inverse * k) {
        stabilizer_contains_eq(a, x, g.inverse * k)
        mul_action_mul(a, g, g.inverse * k, x)
        (g * g.inverse) * k = G.1 * k
        a.act(k, x) = a.act(g * (g.inverse * k), x)
        a.act(g, x) = a.act(k, x)
    }
}

/// An equivariant map sends stabilizer membership forward.
theorem equivariant_map_maps_fixed_point_value[G: Group, X, Y](
    a: MulAction[G, X],
    b: MulAction[G, Y],
    f: X -> Y,
    x: X,
    g: G
) {
    is_equivariant_map(a, b, f) and stabilizer(a, x).contains(g) implies
        b.act(g, f(x)) = f(x)
} by {
    if is_equivariant_map(a, b, f) and stabilizer(a, x).contains(g) {
        stabilizer_fixes(a, x, g)
        equivariant_map_apply(a, b, f, g, x)
        b.act(g, f(x)) = f(x)
    }
}

/// An equivariant map sends stabilizer membership forward.
theorem equivariant_map_maps_stabilizer_contains[G: Group, X, Y](
    a: MulAction[G, X],
    b: MulAction[G, Y],
    f: X -> Y,
    x: X,
    g: G
) {
    is_equivariant_map(a, b, f) and stabilizer(a, x).contains(g) implies
        stabilizer(b, f(x)).contains(g)
} by {
    if is_equivariant_map(a, b, f) and stabilizer(a, x).contains(g) {
        equivariant_map_maps_fixed_point_value(a, b, f, x, g)
        stabilizer_contains_of_fixes(b, f(x), g)
        stabilizer(b, f(x)).contains(g)
    }
}

/// An action homomorphism sends stabilizer membership forward.
theorem action_hom_maps_stabilizer_contains[G: Group, X, Y](
    f: ActionHom[G, X, Y],
    x: X,
    g: G
) {
    stabilizer(f.src, x).contains(g) implies stabilizer(f.dst, f.map(x)).contains(g)
} by {
    if stabilizer(f.src, x).contains(g) {
        action_hom_is_equivariant(f)
        equivariant_map_maps_stabilizer_contains(f.src, f.dst, f.map, x, g)
    }
}

/// An injective equivariant map reflects stabilizer membership.
theorem equivariant_map_reflects_fixed_point_value_of_injective[G: Group, X, Y](
    a: MulAction[G, X],
    b: MulAction[G, Y],
    f: X -> Y,
    x: X,
    g: G
) {
    is_injective_fn(f) and is_equivariant_map(a, b, f) and stabilizer(b, f(x)).contains(g) implies
        a.act(g, x) = x
} by {
    if is_injective_fn(f) and is_equivariant_map(a, b, f) and stabilizer(b, f(x)).contains(g) {
        stabilizer_fixes(b, f(x), g)
        equivariant_map_apply(a, b, f, g, x)
        f(a.act(g, x)) = f(x)
        injective_fn_eq(f, a.act(g, x), x)
        a.act(g, x) = x
    }
}

/// An injective equivariant map reflects stabilizer membership.
theorem equivariant_map_reflects_stabilizer_contains_of_injective[G: Group, X, Y](
    a: MulAction[G, X],
    b: MulAction[G, Y],
    f: X -> Y,
    x: X,
    g: G
) {
    is_injective_fn(f) and is_equivariant_map(a, b, f) and stabilizer(b, f(x)).contains(g) implies
        stabilizer(a, x).contains(g)
} by {
    if is_injective_fn(f) and is_equivariant_map(a, b, f) and stabilizer(b, f(x)).contains(g) {
        equivariant_map_reflects_fixed_point_value_of_injective(a, b, f, x, g)
        stabilizer_contains_of_fixes(a, x, g)
        stabilizer(a, x).contains(g)
    }
}

/// An injective action homomorphism reflects stabilizer membership.
theorem action_hom_reflects_stabilizer_contains_of_injective[G: Group, X, Y](
    f: ActionHom[G, X, Y],
    x: X,
    g: G
) {
    is_injective_fn(f.map) and stabilizer(f.dst, f.map(x)).contains(g) implies
        stabilizer(f.src, x).contains(g)
} by {
    if is_injective_fn(f.map) and stabilizer(f.dst, f.map(x)).contains(g) {
        action_hom_is_equivariant(f)
        equivariant_map_reflects_stabilizer_contains_of_injective(f.src, f.dst, f.map, x, g)
    }
}

/// True if a point is fixed by a group element.
define fixed_points_contains[G: Group, X](a: MulAction[G, X], g: G, x: X) -> Bool {
    a.act(g, x) = x
}

/// The fixed points of a group element.
define fixed_points[G: Group, X](a: MulAction[G, X], g: G) -> Set[X] {
    Set[X].new(fixed_points_contains(a, g))
}

/// Membership in the fixed-point set is being fixed by the element.
theorem fixed_points_contains_eq[G: Group, X](a: MulAction[G, X], g: G, x: X) {
    fixed_points(a, g).contains(x) = (a.act(g, x) = x)
} by {
}

/// The identity element fixes every point.
theorem fixed_points_one_contains[G: Group, X](a: MulAction[G, X], x: X) {
    fixed_points(a, G.1).contains(x)
} by {
    fixed_points_contains_eq(a, G.1, x)
    mul_action_one(a, x)
}

/// True if the action has a single orbit.
define is_transitive_action[G: Group, X](a: MulAction[G, X]) -> Bool {
    forall(x: X, y: X) {
        orbit(a, x).contains(y)
    }
}

/// In a transitive action, every point belongs to every orbit.
theorem transitive_action_orbit_contains[G: Group, X](a: MulAction[G, X], x: X, y: X) {
    is_transitive_action(a) implies orbit(a, x).contains(y)
} by {
    if is_transitive_action(a) {
        is_transitive_action(a) = forall(u: X, v: X) {
            orbit(a, u).contains(v)
        }
    }
}

/// If every orbit is universal, the action is transitive.
theorem transitive_action_of_universal_orbits[G: Group, X](a: MulAction[G, X]) {
    (forall(x: X, y: X) { orbit(a, x).contains(y) }) implies is_transitive_action(a)
} by {
    if forall(x: X, y: X) { orbit(a, x).contains(y) } {
        is_transitive_action(a) = forall(x: X, y: X) {
            orbit(a, x).contains(y)
        }
    }
}

/// True if group elements with the same action on every point are equal.
define is_faithful_action[G: Group, X](a: MulAction[G, X]) -> Bool {
    forall(g: G, h: G) {
        (forall(x: X) { a.act(g, x) = a.act(h, x) }) implies g = h
    }
}

/// A faithful action separates group elements by their pointwise action.
theorem faithful_action_eq_of_pointwise_eq[G: Group, X](a: MulAction[G, X], g: G, h: G) {
    is_faithful_action(a) and (forall(x: X) { a.act(g, x) = a.act(h, x) }) implies g = h
} by {
    if is_faithful_action(a) and forall(x: X) { a.act(g, x) = a.act(h, x) } {
        is_faithful_action(a) = forall(k: G, l: G) {
            (forall(y: X) { a.act(k, y) = a.act(l, y) }) implies k = l
        }
    }
}

/// Pointwise equality of a group element with the identity makes it the identity in a faithful action.
theorem faithful_action_eq_one_of_pointwise_fixed[G: Group, X](a: MulAction[G, X], g: G) {
    is_faithful_action(a) and (forall(x: X) { a.act(g, x) = x }) implies g = G.1
} by {
    if is_faithful_action(a) and forall(x: X) { a.act(g, x) = x } {
        forall(x: X) {
            mul_action_one(a, x)
            a.act(g, x) = a.act(G.1, x)
        }
        faithful_action_eq_of_pointwise_eq(a, g, G.1)
    }
}

/// True if a set is stable under the group action.
define is_invariant_set[G: Group, X](a: MulAction[G, X], s: Set[X]) -> Bool {
    forall(g: G, x: X) {
        s.contains(x) implies s.contains(a.act(g, x))
    }
}

/// The universal set is invariant under every group action.
theorem universal_set_is_invariant[G: Group, X](a: MulAction[G, X]) {
    is_invariant_set(a, Set[X].universal_set)
} by {
    forall(g: G, x: X) {
        if Set[X].universal_set.contains(x) {
            Set[X].universal_set.contains(a.act(g, x))
        }
    }
}

/// Every orbit is invariant under the action.
theorem orbit_is_invariant[G: Group, X](a: MulAction[G, X], x: X) {
    is_invariant_set(a, orbit(a, x))
} by {
    forall(g: G, y: X) {
        if orbit(a, x).contains(y) {
            orbit_contains_witness(a, x, y)
            let h: G satisfy {
                y = a.act(h, x)
            }
            mul_action_mul(a, g, h, x)
            orbit_contains_action(a, x, g * h)
            orbit(a, x).contains(a.act(g, y))
        }
    }
}
