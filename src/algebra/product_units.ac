/// Units for componentwise algebraic operations on binary products.

from pair import Pair, pair_ext
from algebra.zero import Zero
from algebra.monoid.monoid import Monoid
from algebra.group import Group
from algebra.ring.ring import Ring
from algebra.units import Unit, is_monoid_unit, one_is_monoid_unit, unit_val_is_monoid_unit,
    unit_inv_is_monoid_unit, monoid_unit_mul_eq_zero_left, monoid_unit_mul_eq_zero_right,
    monoid_unit_mul_cancel_left, monoid_unit_mul_cancel_right
from algebra.product_algebra import pair_mul, pair_one, pair_zero, pair_inverse,
    pair_mul_inverse_right, pair_mul_inverse_left

/// True if a product element has a two-sided componentwise inverse.
define is_pair_unit[A: Monoid, B: Monoid](p: Pair[A, B]) -> Bool {
    exists(q: Pair[A, B]) {
        pair_mul(p, q) = pair_one[A, B] and pair_mul(q, p) = pair_one[A, B]
    }
}

/// The value pair associated to a pair of units.
define unit_pair_val[A: Monoid, B: Monoid](u: Pair[Unit[A], Unit[B]]) -> Pair[A, B] {
    Pair.new(u.first.val, u.second.val)
}

/// The inverse pair associated to a pair of units.
define unit_pair_inv[A: Monoid, B: Monoid](u: Pair[Unit[A], Unit[B]]) -> Pair[A, B] {
    Pair.new(u.first.inv, u.second.inv)
}

/// The first coordinate of the value pair is the value of the first unit.
theorem unit_pair_val_first[A: Monoid, B: Monoid](u: Pair[Unit[A], Unit[B]]) {
    unit_pair_val(u).first = u.first.val
}

/// The second coordinate of the value pair is the value of the second unit.
theorem unit_pair_val_second[A: Monoid, B: Monoid](u: Pair[Unit[A], Unit[B]]) {
    unit_pair_val(u).second = u.second.val
}

/// The first coordinate of the inverse pair is the inverse of the first unit.
theorem unit_pair_inv_first[A: Monoid, B: Monoid](u: Pair[Unit[A], Unit[B]]) {
    unit_pair_inv(u).first = u.first.inv
}

/// The second coordinate of the inverse pair is the inverse of the second unit.
theorem unit_pair_inv_second[A: Monoid, B: Monoid](u: Pair[Unit[A], Unit[B]]) {
    unit_pair_inv(u).second = u.second.inv
}

/// A pair of unit values has the pair of inverses as a right inverse.
theorem unit_pair_val_mul_inv[A: Monoid, B: Monoid](u: Pair[Unit[A], Unit[B]]) {
    pair_mul(unit_pair_val(u), unit_pair_inv(u)) = pair_one[A, B]
} by {
    let lhs = pair_mul(unit_pair_val(u), unit_pair_inv(u))
    lhs.first = unit_pair_val(u).first * unit_pair_inv(u).first
    unit_pair_val(u).first = u.first.val
    unit_pair_inv(u).first = u.first.inv
    lhs.first = u.first.val * u.first.inv
    u.first.val * u.first.inv = A.1
    lhs.second = unit_pair_val(u).second * unit_pair_inv(u).second
    unit_pair_val(u).second = u.second.val
    unit_pair_inv(u).second = u.second.inv
    lhs.second = u.second.val * u.second.inv
    u.second.val * u.second.inv = B.1
    pair_ext(lhs, pair_one[A, B])
}

/// A pair of unit inverses has the pair of values as a right inverse.
theorem unit_pair_inv_mul_val[A: Monoid, B: Monoid](u: Pair[Unit[A], Unit[B]]) {
    pair_mul(unit_pair_inv(u), unit_pair_val(u)) = pair_one[A, B]
} by {
    let lhs = pair_mul(unit_pair_inv(u), unit_pair_val(u))
    lhs.first = unit_pair_inv(u).first * unit_pair_val(u).first
    unit_pair_inv(u).first = u.first.inv
    unit_pair_val(u).first = u.first.val
    lhs.first = u.first.inv * u.first.val
    u.first.inv * u.first.val = A.1
    lhs.second = unit_pair_inv(u).second * unit_pair_val(u).second
    unit_pair_inv(u).second = u.second.inv
    unit_pair_val(u).second = u.second.val
    lhs.second = u.second.inv * u.second.val
    u.second.inv * u.second.val = B.1
    pair_ext(lhs, pair_one[A, B])
}

/// The value pair associated to a pair of units is a pair unit.
theorem unit_pair_val_is_pair_unit[A: Monoid, B: Monoid](u: Pair[Unit[A], Unit[B]]) {
    is_pair_unit(unit_pair_val(u))
} by {
    exists(q: Pair[A, B]) {
        q = unit_pair_inv(u) and
        pair_mul(unit_pair_val(u), q) = pair_one[A, B] and
        pair_mul(q, unit_pair_val(u)) = pair_one[A, B]
    }
}

/// A pair whose coordinates are unit values is a pair unit.
theorem pair_of_unit_values_is_pair_unit[A: Monoid, B: Monoid](u: Unit[A], v: Unit[B]) {
    is_pair_unit(Pair.new(u.val, v.val))
} by {
    let p = Pair.new(u, v)
    unit_pair_val_is_pair_unit(p)
    let q = unit_pair_val(p)
    q.first = p.first.val
    p.first = u
    q.first = u.val
    q.second = p.second.val
    p.second = v
    q.second = v.val
    pair_ext(q, Pair.new(u.val, v.val))
    is_pair_unit(Pair.new(u.val, v.val))
}

/// The first coordinate of a pair unit is a unit.
theorem pair_unit_first_is_monoid_unit[A: Monoid, B: Monoid](p: Pair[A, B]) {
    is_pair_unit(p) implies is_monoid_unit(p.first)
} by {
    if is_pair_unit(p) {
        let q: Pair[A, B] satisfy {
            pair_mul(p, q) = pair_one[A, B] and pair_mul(q, p) = pair_one[A, B]
        }
        pair_mul(p, q).first = p.first * q.first
        pair_one[A, B].first = A.1
        p.first * q.first = A.1
        pair_mul(q, p).first = q.first * p.first
        q.first * p.first = A.1
        exists(a: A) {
            a = q.first and p.first * a = A.1 and a * p.first = A.1
        }
    }
}

/// The second coordinate of a pair unit is a unit.
theorem pair_unit_second_is_monoid_unit[A: Monoid, B: Monoid](p: Pair[A, B]) {
    is_pair_unit(p) implies is_monoid_unit(p.second)
} by {
    if is_pair_unit(p) {
        let q: Pair[A, B] satisfy {
            pair_mul(p, q) = pair_one[A, B] and pair_mul(q, p) = pair_one[A, B]
        }
        pair_mul(p, q).second = p.second * q.second
        pair_one[A, B].second = B.1
        p.second * q.second = B.1
        pair_mul(q, p).second = q.second * p.second
        q.second * p.second = B.1
        exists(b: B) {
            b = q.second and p.second * b = B.1 and b * p.second = B.1
        }
    }
}

/// A product whose coordinates are units is a pair unit.
theorem pair_unit_of_components[A: Monoid, B: Monoid](p: Pair[A, B]) {
    is_monoid_unit(p.first) and is_monoid_unit(p.second) implies is_pair_unit(p)
} by {
    if is_monoid_unit(p.first) and is_monoid_unit(p.second) {
        let a: A satisfy {
            p.first * a = A.1 and a * p.first = A.1
        }
        let b: B satisfy {
            p.second * b = B.1 and b * p.second = B.1
        }
        let q = Pair.new(a, b)
        pair_mul(p, q).first = p.first * q.first
        q.first = a
        pair_mul(p, q).first = p.first * a
        pair_mul(p, q).first = A.1
        pair_mul(p, q).second = p.second * q.second
        q.second = b
        pair_mul(p, q).second = p.second * b
        pair_mul(p, q).second = B.1
        pair_mul(p, q) = pair_one[A, B]
        pair_mul(q, p).first = q.first * p.first
        pair_mul(q, p).first = a * p.first
        pair_mul(q, p).first = A.1
        pair_mul(q, p).second = q.second * p.second
        pair_mul(q, p).second = b * p.second
        pair_mul(q, p).second = B.1
        pair_mul(q, p) = pair_one[A, B]
        exists(r: Pair[A, B]) {
            r = q and pair_mul(p, r) = pair_one[A, B] and pair_mul(r, p) = pair_one[A, B]
        }
    }
}

/// A product element is a unit exactly when both coordinates are units.
theorem pair_unit_iff_components[A: Monoid, B: Monoid](p: Pair[A, B]) {
    is_pair_unit(p) = (is_monoid_unit(p.first) and is_monoid_unit(p.second))
} by {
    if is_pair_unit(p) {
        pair_unit_first_is_monoid_unit(p)
        pair_unit_second_is_monoid_unit(p)
        is_monoid_unit(p.first) and is_monoid_unit(p.second)
    }
    if is_monoid_unit(p.first) and is_monoid_unit(p.second) {
        pair_unit_of_components(p)
        is_pair_unit(p)
    }
}

/// The componentwise identity is a pair unit.
theorem pair_one_is_pair_unit[A: Monoid, B: Monoid] {
    is_pair_unit(pair_one[A, B])
} by {
    one_is_monoid_unit[A]
    one_is_monoid_unit[B]
    pair_one[A, B].first = A.1
    pair_one[A, B].second = B.1
    is_monoid_unit(pair_one[A, B].first)
    is_monoid_unit(pair_one[A, B].second)
    pair_unit_of_components(pair_one[A, B])
}

/// The componentwise product of two pair units is a pair unit.
theorem pair_unit_mul[A: Monoid, B: Monoid](p: Pair[A, B], q: Pair[A, B]) {
    is_pair_unit(p) and is_pair_unit(q) implies is_pair_unit(pair_mul(p, q))
} by {
    if is_pair_unit(p) and is_pair_unit(q) {
        let pi: Pair[A, B] satisfy {
            pair_mul(p, pi) = pair_one[A, B] and pair_mul(pi, p) = pair_one[A, B]
        }
        let qi: Pair[A, B] satisfy {
            pair_mul(q, qi) = pair_one[A, B] and pair_mul(qi, q) = pair_one[A, B]
        }
        let r = pair_mul(qi, pi)
        let lhs = pair_mul(pair_mul(p, q), r)
        lhs.first = pair_mul(p, q).first * r.first
        pair_mul(p, q).first = p.first * q.first
        r.first = qi.first * pi.first
        lhs.first = (p.first * q.first) * (qi.first * pi.first)
        (p.first * q.first) * (qi.first * pi.first) = p.first * (q.first * qi.first) * pi.first
        pair_mul(q, qi).first = q.first * qi.first
        pair_one[A, B].first = A.1
        q.first * qi.first = A.1
        lhs.first = p.first * A.1 * pi.first
        p.first * A.1 = p.first
        lhs.first = p.first * pi.first
        pair_mul(p, pi).first = p.first * pi.first
        p.first * pi.first = A.1
        lhs.first = A.1
        lhs.second = pair_mul(p, q).second * r.second
        pair_mul(p, q).second = p.second * q.second
        r.second = qi.second * pi.second
        lhs.second = (p.second * q.second) * (qi.second * pi.second)
        (p.second * q.second) * (qi.second * pi.second) = p.second * (q.second * qi.second) * pi.second
        pair_mul(q, qi).second = q.second * qi.second
        pair_one[A, B].second = B.1
        q.second * qi.second = B.1
        lhs.second = p.second * B.1 * pi.second
        p.second * B.1 = p.second
        lhs.second = p.second * pi.second
        pair_mul(p, pi).second = p.second * pi.second
        p.second * pi.second = B.1
        lhs.second = B.1
        pair_mul(pair_mul(p, q), r) = pair_one[A, B]

        let rhs = pair_mul(r, pair_mul(p, q))
        rhs.first = r.first * pair_mul(p, q).first
        r.first = qi.first * pi.first
        pair_mul(p, q).first = p.first * q.first
        rhs.first = (qi.first * pi.first) * (p.first * q.first)
        (qi.first * pi.first) * (p.first * q.first) = qi.first * (pi.first * p.first) * q.first
        pair_mul(pi, p).first = pi.first * p.first
        pi.first * p.first = A.1
        rhs.first = qi.first * A.1 * q.first
        qi.first * A.1 = qi.first
        rhs.first = qi.first * q.first
        pair_mul(qi, q).first = qi.first * q.first
        qi.first * q.first = A.1
        rhs.first = A.1
        rhs.second = r.second * pair_mul(p, q).second
        r.second = qi.second * pi.second
        pair_mul(p, q).second = p.second * q.second
        rhs.second = (qi.second * pi.second) * (p.second * q.second)
        (qi.second * pi.second) * (p.second * q.second) = qi.second * (pi.second * p.second) * q.second
        pair_mul(pi, p).second = pi.second * p.second
        pi.second * p.second = B.1
        rhs.second = qi.second * B.1 * q.second
        qi.second * B.1 = qi.second
        rhs.second = qi.second * q.second
        pair_mul(qi, q).second = qi.second * q.second
        qi.second * q.second = B.1
        rhs.second = B.1
        pair_mul(r, pair_mul(p, q)) = pair_one[A, B]

        exists(s: Pair[A, B]) {
            s = r and
            pair_mul(pair_mul(p, q), s) = pair_one[A, B] and
            pair_mul(s, pair_mul(p, q)) = pair_one[A, B]
        }
    }
}

/// A stated two-sided componentwise inverse makes the original element a pair unit.
theorem pair_unit_of_inverse[A: Monoid, B: Monoid](p: Pair[A, B], q: Pair[A, B]) {
    pair_mul(p, q) = pair_one[A, B] and pair_mul(q, p) = pair_one[A, B]
    implies is_pair_unit(p)
} by {
    if pair_mul(p, q) = pair_one[A, B] and pair_mul(q, p) = pair_one[A, B] {
        exists(r: Pair[A, B]) {
            r = q and pair_mul(p, r) = pair_one[A, B] and pair_mul(r, p) = pair_one[A, B]
        }
    }
}

/// A stated two-sided componentwise inverse is itself a pair unit.
theorem pair_inverse_witness_is_pair_unit[A: Monoid, B: Monoid](p: Pair[A, B], q: Pair[A, B]) {
    pair_mul(p, q) = pair_one[A, B] and pair_mul(q, p) = pair_one[A, B]
    implies is_pair_unit(q)
} by {
    if pair_mul(p, q) = pair_one[A, B] and pair_mul(q, p) = pair_one[A, B] {
        exists(r: Pair[A, B]) {
            r = p and pair_mul(q, r) = pair_one[A, B] and pair_mul(r, q) = pair_one[A, B]
        }
    }
}

/// A pair unit witness has unit coordinates.
theorem pair_unit_witness_components[A: Monoid, B: Monoid](p: Pair[A, B], q: Pair[A, B]) {
    pair_mul(p, q) = pair_one[A, B] and pair_mul(q, p) = pair_one[A, B]
    implies is_monoid_unit(p.first) and is_monoid_unit(p.second)
} by {
    if pair_mul(p, q) = pair_one[A, B] and pair_mul(q, p) = pair_one[A, B] {
        pair_unit_of_inverse(p, q)
        pair_unit_first_is_monoid_unit(p)
        pair_unit_second_is_monoid_unit(p)
    }
}

/// The componentwise inverse witness of a pair of units is a pair unit.
theorem unit_pair_inv_is_pair_unit[A: Monoid, B: Monoid](u: Pair[Unit[A], Unit[B]]) {
    is_pair_unit(unit_pair_inv(u))
} by {
    unit_pair_val_mul_inv(u)
    unit_pair_inv_mul_val(u)
    pair_inverse_witness_is_pair_unit(unit_pair_val(u), unit_pair_inv(u))
}

/// Each coordinate of a pair of unit values is a monoid unit.
theorem unit_pair_val_components_are_units[A: Monoid, B: Monoid](u: Pair[Unit[A], Unit[B]]) {
    is_monoid_unit(unit_pair_val(u).first) and is_monoid_unit(unit_pair_val(u).second)
} by {
    unit_pair_val(u).first = u.first.val
    unit_pair_val(u).second = u.second.val
    unit_val_is_monoid_unit(u.first)
    unit_val_is_monoid_unit(u.second)
}

/// Each coordinate of a pair of unit inverses is a monoid unit.
theorem unit_pair_inv_components_are_units[A: Monoid, B: Monoid](u: Pair[Unit[A], Unit[B]]) {
    is_monoid_unit(unit_pair_inv(u).first) and is_monoid_unit(unit_pair_inv(u).second)
} by {
    unit_pair_inv(u).first = u.first.inv
    unit_pair_inv(u).second = u.second.inv
    unit_inv_is_monoid_unit(u.first)
    unit_inv_is_monoid_unit(u.second)
}

/// Every element of a product of groups is a pair unit.
theorem pair_unit_of_group[A: Group, B: Group](p: Pair[A, B]) {
    is_pair_unit(p)
} by {
    pair_mul_inverse_right(p)
    pair_mul_inverse_left(p)
    exists(q: Pair[A, B]) {
        q = pair_inverse(p) and
        pair_mul(p, q) = pair_one[A, B] and
        pair_mul(q, p) = pair_one[A, B]
    }
}

/// The componentwise inverse of an element of a product of groups is a pair unit.
theorem pair_inverse_is_pair_unit[A: Group, B: Group](p: Pair[A, B]) {
    is_pair_unit(pair_inverse(p))
} by {
    pair_mul_inverse_right(p)
    pair_mul_inverse_left(p)
    pair_inverse_witness_is_pair_unit(p, pair_inverse(p))
}

/// Multiplication by a pair unit on the left has trivial first-coordinate zero kernel.
theorem pair_unit_mul_eq_zero_left_first[A: Ring, B: Ring](u: Pair[A, B], p: Pair[A, B]) {
    is_pair_unit(u) and pair_mul(u, p) = pair_zero[A, B] implies p.first = A.0
} by {
    if is_pair_unit(u) and pair_mul(u, p) = pair_zero[A, B] {
        pair_unit_first_is_monoid_unit(u)
        pair_mul(u, p).first = u.first * p.first
        pair_zero[A, B].first = A.0
        u.first * p.first = A.0
        monoid_unit_mul_eq_zero_left(u.first, p.first)
        p.first = A.0
    }
}

/// Multiplication by a pair unit on the left has trivial second-coordinate zero kernel.
theorem pair_unit_mul_eq_zero_left_second[A: Ring, B: Ring](u: Pair[A, B], p: Pair[A, B]) {
    is_pair_unit(u) and pair_mul(u, p) = pair_zero[A, B] implies p.second = B.0
} by {
    if is_pair_unit(u) and pair_mul(u, p) = pair_zero[A, B] {
        pair_unit_second_is_monoid_unit(u)
        pair_mul(u, p).second = u.second * p.second
        pair_zero[A, B].second = B.0
        u.second * p.second = B.0
        monoid_unit_mul_eq_zero_left(u.second, p.second)
        p.second = B.0
    }
}

/// Multiplication by a pair unit on the right has trivial first-coordinate zero kernel.
theorem pair_unit_mul_eq_zero_right_first[A: Ring, B: Ring](u: Pair[A, B], p: Pair[A, B]) {
    is_pair_unit(u) and pair_mul(p, u) = pair_zero[A, B] implies p.first = A.0
} by {
    if is_pair_unit(u) and pair_mul(p, u) = pair_zero[A, B] {
        pair_unit_first_is_monoid_unit(u)
        pair_mul(p, u).first = p.first * u.first
        pair_zero[A, B].first = A.0
        p.first * u.first = A.0
        monoid_unit_mul_eq_zero_right(u.first, p.first)
        p.first = A.0
    }
}

/// Multiplication by a pair unit on the right has trivial second-coordinate zero kernel.
theorem pair_unit_mul_eq_zero_right_second[A: Ring, B: Ring](u: Pair[A, B], p: Pair[A, B]) {
    is_pair_unit(u) and pair_mul(p, u) = pair_zero[A, B] implies p.second = B.0
} by {
    if is_pair_unit(u) and pair_mul(p, u) = pair_zero[A, B] {
        pair_unit_second_is_monoid_unit(u)
        pair_mul(p, u).second = p.second * u.second
        pair_zero[A, B].second = B.0
        p.second * u.second = B.0
        monoid_unit_mul_eq_zero_right(u.second, p.second)
        p.second = B.0
    }
}

/// Multiplication by a pair unit on the left is cancellative in the first coordinate.
theorem pair_unit_mul_cancel_left_first[A: Ring, B: Ring](
    u: Pair[A, B],
    p: Pair[A, B],
    q: Pair[A, B]
) {
    is_pair_unit(u) and pair_mul(u, p) = pair_mul(u, q) implies p.first = q.first
} by {
    if is_pair_unit(u) and pair_mul(u, p) = pair_mul(u, q) {
        pair_unit_first_is_monoid_unit(u)
        pair_mul(u, p).first = u.first * p.first
        pair_mul(u, q).first = u.first * q.first
        u.first * p.first = u.first * q.first
        monoid_unit_mul_cancel_left(u.first, p.first, q.first)
        p.first = q.first
    }
}

/// Multiplication by a pair unit on the left is cancellative in the second coordinate.
theorem pair_unit_mul_cancel_left_second[A: Ring, B: Ring](
    u: Pair[A, B],
    p: Pair[A, B],
    q: Pair[A, B]
) {
    is_pair_unit(u) and pair_mul(u, p) = pair_mul(u, q) implies p.second = q.second
} by {
    if is_pair_unit(u) and pair_mul(u, p) = pair_mul(u, q) {
        pair_unit_second_is_monoid_unit(u)
        pair_mul(u, p).second = u.second * p.second
        pair_mul(u, q).second = u.second * q.second
        u.second * p.second = u.second * q.second
        monoid_unit_mul_cancel_left(u.second, p.second, q.second)
        p.second = q.second
    }
}

/// Multiplication by a pair unit on the right is cancellative in the first coordinate.
theorem pair_unit_mul_cancel_right_first[A: Ring, B: Ring](
    u: Pair[A, B],
    p: Pair[A, B],
    q: Pair[A, B]
) {
    is_pair_unit(u) and pair_mul(p, u) = pair_mul(q, u) implies p.first = q.first
} by {
    if is_pair_unit(u) and pair_mul(p, u) = pair_mul(q, u) {
        pair_unit_first_is_monoid_unit(u)
        pair_mul(p, u).first = p.first * u.first
        pair_mul(q, u).first = q.first * u.first
        p.first * u.first = q.first * u.first
        monoid_unit_mul_cancel_right(u.first, p.first, q.first)
        p.first = q.first
    }
}

/// Multiplication by a pair unit on the right is cancellative in the second coordinate.
theorem pair_unit_mul_cancel_right_second[A: Ring, B: Ring](
    u: Pair[A, B],
    p: Pair[A, B],
    q: Pair[A, B]
) {
    is_pair_unit(u) and pair_mul(p, u) = pair_mul(q, u) implies p.second = q.second
} by {
    if is_pair_unit(u) and pair_mul(p, u) = pair_mul(q, u) {
        pair_unit_second_is_monoid_unit(u)
        pair_mul(p, u).second = p.second * u.second
        pair_mul(q, u).second = q.second * u.second
        p.second * u.second = q.second * u.second
        monoid_unit_mul_cancel_right(u.second, p.second, q.second)
        p.second = q.second
    }
}

/// A product is zero when both coordinates are zero.
theorem pair_eq_zero_of_components_zero[A: Zero, B: Zero](p: Pair[A, B]) {
    p.first = A.0 and p.second = B.0 implies p = pair_zero[A, B]
} by {
    if p.first = A.0 and p.second = B.0 {
        pair_zero[A, B].first = A.0
        pair_zero[A, B].second = B.0
        p.first = pair_zero[A, B].first
        p.second = pair_zero[A, B].second
        p.first = pair_zero[A, B].first and p.second = pair_zero[A, B].second
        pair_ext(p, pair_zero[A, B])
    }
}

/// Two products are equal when corresponding coordinates are equal.
theorem pair_eq_of_components_eq[A, B](p: Pair[A, B], q: Pair[A, B]) {
    p.first = q.first and p.second = q.second implies p = q
} by {
    if p.first = q.first and p.second = q.second {
        pair_ext(p, q)
    }
}

/// Multiplication by a pair unit on the left has trivial zero kernel in both coordinates.
theorem pair_unit_mul_eq_zero_left_components[A: Ring, B: Ring](u: Pair[A, B], p: Pair[A, B]) {
    is_pair_unit(u) and pair_mul(u, p) = pair_zero[A, B] implies
    p.first = A.0 and p.second = B.0
} by {
    if is_pair_unit(u) and pair_mul(u, p) = pair_zero[A, B] {
        pair_unit_mul_eq_zero_left_first(u, p)
        pair_unit_mul_eq_zero_left_second(u, p)
        p.first = A.0
        p.second = B.0
        p.first = A.0 and p.second = B.0
    }
}

/// Multiplication by a pair unit on the left has trivial zero kernel.
theorem pair_unit_mul_eq_zero_left[A: Ring, B: Ring](u: Pair[A, B], p: Pair[A, B]) {
    is_pair_unit(u) and pair_mul(u, p) = pair_zero[A, B] implies p = pair_zero[A, B]
} by {
    if is_pair_unit(u) and pair_mul(u, p) = pair_zero[A, B] {
        pair_unit_mul_eq_zero_left_components(u, p)
        p.first = A.0 and p.second = B.0
        pair_eq_zero_of_components_zero(p)
        p = pair_zero[A, B]
    }
}

/// Multiplication by a pair unit on the right has trivial zero kernel in both coordinates.
theorem pair_unit_mul_eq_zero_right_components[A: Ring, B: Ring](u: Pair[A, B], p: Pair[A, B]) {
    is_pair_unit(u) and pair_mul(p, u) = pair_zero[A, B] implies
    p.first = A.0 and p.second = B.0
} by {
    if is_pair_unit(u) and pair_mul(p, u) = pair_zero[A, B] {
        pair_unit_mul_eq_zero_right_first(u, p)
        pair_unit_mul_eq_zero_right_second(u, p)
        p.first = A.0
        p.second = B.0
        p.first = A.0 and p.second = B.0
    }
}

/// Multiplication by a pair unit on the right has trivial zero kernel.
theorem pair_unit_mul_eq_zero_right[A: Ring, B: Ring](u: Pair[A, B], p: Pair[A, B]) {
    is_pair_unit(u) and pair_mul(p, u) = pair_zero[A, B] implies p = pair_zero[A, B]
} by {
    if is_pair_unit(u) and pair_mul(p, u) = pair_zero[A, B] {
        pair_unit_mul_eq_zero_right_components(u, p)
        p.first = A.0 and p.second = B.0
        pair_eq_zero_of_components_zero(p)
        p = pair_zero[A, B]
    }
}

/// Multiplication by a pair unit on the left is cancellative in both coordinates.
theorem pair_unit_mul_cancel_left_components[A: Ring, B: Ring](
    u: Pair[A, B],
    p: Pair[A, B],
    q: Pair[A, B]
) {
    is_pair_unit(u) and pair_mul(u, p) = pair_mul(u, q) implies
    p.first = q.first and p.second = q.second
} by {
    if is_pair_unit(u) and pair_mul(u, p) = pair_mul(u, q) {
        pair_unit_mul_cancel_left_first(u, p, q)
        pair_unit_mul_cancel_left_second(u, p, q)
        p.first = q.first
        p.second = q.second
        p.first = q.first and p.second = q.second
    }
}

/// Multiplication by a pair unit on the left is cancellative.
theorem pair_unit_mul_cancel_left[A: Ring, B: Ring](
    u: Pair[A, B],
    p: Pair[A, B],
    q: Pair[A, B]
) {
    is_pair_unit(u) and pair_mul(u, p) = pair_mul(u, q) implies p = q
} by {
    if is_pair_unit(u) and pair_mul(u, p) = pair_mul(u, q) {
        pair_unit_mul_cancel_left_components(u, p, q)
        p.first = q.first and p.second = q.second
        pair_eq_of_components_eq(p, q)
        p = q
    }
}

/// Multiplication by a pair unit on the right is cancellative in both coordinates.
theorem pair_unit_mul_cancel_right_components[A: Ring, B: Ring](
    u: Pair[A, B],
    p: Pair[A, B],
    q: Pair[A, B]
) {
    is_pair_unit(u) and pair_mul(p, u) = pair_mul(q, u) implies
    p.first = q.first and p.second = q.second
} by {
    if is_pair_unit(u) and pair_mul(p, u) = pair_mul(q, u) {
        pair_unit_mul_cancel_right_first(u, p, q)
        pair_unit_mul_cancel_right_second(u, p, q)
        p.first = q.first
        p.second = q.second
        p.first = q.first and p.second = q.second
    }
}

/// Multiplication by a pair unit on the right is cancellative.
theorem pair_unit_mul_cancel_right[A: Ring, B: Ring](
    u: Pair[A, B],
    p: Pair[A, B],
    q: Pair[A, B]
) {
    is_pair_unit(u) and pair_mul(p, u) = pair_mul(q, u) implies p = q
} by {
    if is_pair_unit(u) and pair_mul(p, u) = pair_mul(q, u) {
        pair_unit_mul_cancel_right_components(u, p, q)
        p.first = q.first and p.second = q.second
        pair_eq_of_components_eq(p, q)
        p = q
    }
}
