from algebra.add_comm_monoid import AddCommMonoid

/// Swapping the inner two summands of a sum of two sums.
///
/// The regrouping that turns a fact about each of two sums into a fact about their total. It
/// appears wherever two additive structures are combined termwise, and proof search does not
/// find the associativity and commutativity chain on its own.
theorem add_swap_inner[A: AddCommMonoid](a: A, b: A, c: A, d: A) {
    (a + b) + (c + d) = (a + c) + (b + d)
} by {
    a + (b + (c + d)) = (a + b) + (c + d)
    b + (c + d) = (b + c) + d
    b + c = c + b
    (b + c) + d = (c + b) + d
    c + (b + d) = (c + b) + d
    b + (c + d) = c + (b + d)
    a + (b + (c + d)) = a + (c + (b + d))
    a + (c + (b + d)) = (a + c) + (b + d)
    (a + b) + (c + d) = (a + c) + (b + d)
}

/// Moving the outer left summand inside on the right.
theorem add_shift_left[A: AddCommMonoid](a: A, b: A, c: A) {
    (a + b) + c = (a + c) + b
} by {
    a + (b + c) = (a + b) + c
    b + c = c + b
    a + (b + c) = a + (c + b)
    a + (c + b) = (a + c) + b
}

/// Regrouping a sum of three so the last two associate together.
///
/// Stated on its own because the step appears inside inductions where the surrounding goal is
/// already large enough that search will not find it.
theorem add_assoc_right[A: AddCommMonoid](a: A, b: A, c: A) {
    (a + b) + c = a + (b + c)
} by {
    a + (b + c) = (a + b) + c
}
