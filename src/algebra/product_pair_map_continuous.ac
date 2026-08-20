/// Continuity of componentwise maps between product topologies.

from data.basic.set import Set, set_ext, set_preimage, set_preimage_contains_eq,
    preimage_contains, set_preimage_monotone
from pair import Pair, pair_map, pair_map_first_eq, pair_map_second_eq
from analysis import TopologicalSpace, box, box_contains_eq, is_continuous,
    is_neighborhood, is_box, generated_open, generated_open_witness,
    is_open_in_product, is_open_in_product_imp_generated_open,
    open_iff_neighborhood_of_each_point, continuous_open_preimage
from algebra.product_open_axioms import pair_is_open_eq, box_open

/// The preimage of a product box under a componentwise map is the product box of
/// the two factor preimages.
theorem pair_map_preimage_box[X, Y, Z, W](
    f: X -> Z, g: Y -> W, u: Set[Z], v: Set[W]) {
    set_preimage(pair_map[X, Y, Z, W](f, g), box(u, v))
        = box(set_preimage(f, u), set_preimage(g, v))
} by {
    let h = pair_map[X, Y, Z, W](f, g)
    let l = set_preimage(f, u)
    let r = set_preimage(g, v)
    forall(p: Pair[X, Y]) {
        pair_map_first_eq[X, Y, Z, W](f, g, p)
        pair_map_second_eq[X, Y, Z, W](f, g, p)
        h(p).first = f(p.first)
        h(p).second = g(p.second)
        set_preimage_contains_eq(h, box(u, v), p)
        preimage_contains(h, box(u, v), p) = box(u, v).contains(h(p))
        box_contains_eq(u, v, h(p))
        box(u, v).contains(h(p)) = (u.contains(h(p).first) and v.contains(h(p).second))
        box(u, v).contains(h(p)) = (u.contains(f(p.first)) and v.contains(g(p.second)))
        set_preimage_contains_eq(f, u, p.first)
        preimage_contains(f, u, p.first) = u.contains(f(p.first))
        set_preimage_contains_eq(g, v, p.second)
        preimage_contains(g, v, p.second) = v.contains(g(p.second))
        box_contains_eq(l, r, p)
        box(l, r).contains(p) = (l.contains(p.first) and r.contains(p.second))
        box(l, r).contains(p) = (u.contains(f(p.first)) and v.contains(g(p.second)))
        set_preimage(h, box(u, v)).contains(p) = box(l, r).contains(p)
    }
    set_ext(set_preimage(h, box(u, v)), box(l, r))
}

/// The preimage of an open box under a componentwise map of continuous maps is
/// open in the source product topology.
theorem pair_map_preimage_box_open[X: TopologicalSpace, Y: TopologicalSpace, Z: TopologicalSpace, W: TopologicalSpace](
    f: X -> Z, g: Y -> W, u: Set[Z], v: Set[W]) {
    is_continuous(f) and is_continuous(g) and Z.is_open(u) and W.is_open(v)
        implies Pair[X, Y].is_open(set_preimage(pair_map[X, Y, Z, W](f, g), box(u, v)))
} by {
    if is_continuous(f) and is_continuous(g) and Z.is_open(u) and W.is_open(v) {
        is_continuous(f)
        is_continuous(g)
        Z.is_open(u)
        W.is_open(v)
        is_continuous(f) and Z.is_open(u)
        continuous_open_preimage[X, Z](f, u)
        X.is_open(set_preimage(f, u))
        is_continuous(g) and W.is_open(v)
        continuous_open_preimage[Y, W](g, v)
        Y.is_open(set_preimage(g, v))
        box_open[X, Y](set_preimage(f, u), set_preimage(g, v))
        Pair[X, Y].is_open(box(set_preimage(f, u), set_preimage(g, v)))
        pair_map_preimage_box[X, Y, Z, W](f, g, u, v)
        set_preimage(pair_map[X, Y, Z, W](f, g), box(u, v)) = box(set_preimage(f, u), set_preimage(g, v))
        Pair[X, Y].is_open(set_preimage(pair_map[X, Y, Z, W](f, g), box(u, v)))
    }
}

/// Around a point in the preimage of a product-open set, a componentwise map has
/// an open basic-box preimage contained in that product-open preimage.
theorem pair_map_preimage_product_open_neighborhood[X: TopologicalSpace, Y: TopologicalSpace, Z: TopologicalSpace, W: TopologicalSpace](
    f: X -> Z, g: Y -> W, s: Set[Pair[Z, W]], p: Pair[X, Y]
) {
    is_continuous(f) and is_continuous(g)
    and Pair[Z, W].is_open(s)
    and set_preimage(pair_map[X, Y, Z, W](f, g), s).contains(p)
    implies is_neighborhood(set_preimage(pair_map[X, Y, Z, W](f, g), s), p)
} by {
    if is_continuous(f) {
        if is_continuous(g) {
            if Pair[Z, W].is_open(s) {
                if set_preimage(pair_map[X, Y, Z, W](f, g), s).contains(p) {
                    let h = pair_map[X, Y, Z, W](f, g)
                    pair_is_open_eq[Z, W](s)
                    Pair[Z, W].is_open(s) = is_open_in_product[Z, W](s)
                    is_open_in_product[Z, W](s)
                    is_open_in_product_imp_generated_open[Z, W](s)
                    generated_open(is_box[Z, W], s)
                    set_preimage_contains_eq(h, s, p)
                    preimage_contains(h, s, p) = s.contains(h(p))
                    s.contains(h(p))
                    generated_open_witness(is_box[Z, W], s, h(p))
                    let b: Set[Pair[Z, W]] satisfy {
                        is_box[Z, W](b) and b.contains(h(p)) and b.subset(s)
                    }
                    is_box[Z, W](b) = exists(u: Set[Z], v: Set[W]) {
                        Z.is_open(u) and W.is_open(v) and b = box(u, v)
                    }
                    let u: Set[Z] satisfy {
                        exists(v: Set[W]) {
                            Z.is_open(u) and W.is_open(v) and b = box(u, v)
                        }
                    }
                    let v: Set[W] satisfy {
                        Z.is_open(u) and W.is_open(v) and b = box(u, v)
                    }
                    Z.is_open(u)
                    W.is_open(v)
                    b = box(u, v)
                    pair_map_preimage_box_open[X, Y, Z, W](f, g, u, v)
                    Pair[X, Y].is_open(set_preimage(pair_map[X, Y, Z, W](f, g), box(u, v)))
                    Pair[X, Y].is_open(set_preimage(h, box(u, v)))
                    Pair[X, Y].is_open(set_preimage(h, b))
                    set_preimage_contains_eq(h, b, p)
                    preimage_contains(h, b, p) = b.contains(h(p))
                    set_preimage(h, b).contains(p)
                    set_preimage_monotone(h, b, s)
                    set_preimage(h, b).subset(set_preimage(h, s))
                    Pair[X, Y].is_open(set_preimage(h, b)) and set_preimage(h, b).contains(p) and set_preimage(h, b).subset(set_preimage(h, s))
                    exists(n: Set[Pair[X, Y]]) {
                        Pair[X, Y].is_open(n) and n.contains(p) and n.subset(set_preimage(h, s))
                    }
                    is_neighborhood(set_preimage(h, s), p)
                    is_neighborhood(set_preimage(pair_map[X, Y, Z, W](f, g), s), p)
                }
            }
        }
    }
}

/// The preimage of a product-open set under a componentwise map of continuous
/// maps is open.
theorem pair_map_preimage_product_open[X: TopologicalSpace, Y: TopologicalSpace, Z: TopologicalSpace, W: TopologicalSpace](
    f: X -> Z, g: Y -> W, s: Set[Pair[Z, W]]
) {
    is_continuous(f) and is_continuous(g) and Pair[Z, W].is_open(s)
        implies Pair[X, Y].is_open(set_preimage(pair_map[X, Y, Z, W](f, g), s))
} by {
    let h = pair_map[X, Y, Z, W](f, g)
    let n = set_preimage(h, s)
    if is_continuous(f) and is_continuous(g) and Pair[Z, W].is_open(s) {
        open_iff_neighborhood_of_each_point[Pair[X, Y]](n)
        Pair[X, Y].is_open(n) = forall(p: Pair[X, Y]) {
            n.contains(p) implies is_neighborhood(n, p)
        }
        let pred: Pair[X, Y] -> Bool = function(p: Pair[X, Y]) {
            n.contains(p) implies is_neighborhood(n, p)
        }
        pred = function(p: Pair[X, Y]) {
            n.contains(p) implies is_neighborhood(n, p)
        }
        forall(p: Pair[X, Y]) {
            if n.contains(p) {
                n = set_preimage(pair_map[X, Y, Z, W](f, g), s)
                set_preimage(pair_map[X, Y, Z, W](f, g), s).contains(p)
                is_continuous(f) and is_continuous(g) and Pair[Z, W].is_open(s) and set_preimage(pair_map[X, Y, Z, W](f, g), s).contains(p)
                pair_map_preimage_product_open_neighborhood[X, Y, Z, W](f, g, s, p)
                is_neighborhood(set_preimage(pair_map[X, Y, Z, W](f, g), s), p)
                n = set_preimage(pair_map[X, Y, Z, W](f, g), s)
                is_neighborhood(n, p)
                function(q: Pair[X, Y]) {
                    n.contains(q) implies is_neighborhood(n, q)
                }(p)
                pred(p)
            }
        }
        forall(p: Pair[X, Y]) {
            pred(p)
        }
        Pair[X, Y].is_open(n)
        n = set_preimage(pair_map[X, Y, Z, W](f, g), s)
        Pair[X, Y].is_open(set_preimage(pair_map[X, Y, Z, W](f, g), s))
    }
}

/// A componentwise map between product spaces is continuous when both component
/// maps are continuous.
theorem pair_map_continuous[X: TopologicalSpace, Y: TopologicalSpace, Z: TopologicalSpace, W: TopologicalSpace](
    f: X -> Z, g: Y -> W
) {
    is_continuous(f) and is_continuous(g)
        implies is_continuous(pair_map[X, Y, Z, W](f, g))
} by {
    if is_continuous(f) and is_continuous(g) {
        forall(s: Set[Pair[Z, W]]) {
            if Pair[Z, W].is_open(s) {
                pair_map_preimage_product_open(f, g, s)
                Pair[X, Y].is_open(set_preimage(pair_map[X, Y, Z, W](f, g), s))
            }
        }
        is_continuous(pair_map[X, Y, Z, W](f, g)) = forall(s: Set[Pair[Z, W]]) {
            Pair[Z, W].is_open(s) implies Pair[X, Y].is_open(set_preimage(pair_map[X, Y, Z, W](f, g), s))
        }
        let pred: Set[Pair[Z, W]] -> Bool = function(s: Set[Pair[Z, W]]) {
            Pair[Z, W].is_open(s) implies Pair[X, Y].is_open(set_preimage(pair_map[X, Y, Z, W](f, g), s))
        }
        forall(s: Set[Pair[Z, W]]) {
            if Pair[Z, W].is_open(s) {
                is_continuous(f) and is_continuous(g) and Pair[Z, W].is_open(s)
                pair_map_preimage_product_open[X, Y, Z, W](f, g, s)
                Pair[X, Y].is_open(set_preimage(pair_map[X, Y, Z, W](f, g), s))
                pred(s)
            } else {
                pred(s)
            }
        }
        forall(s: Set[Pair[Z, W]]) {
            pred(s)
        }
        is_continuous(pair_map[X, Y, Z, W](f, g))
    }
}
