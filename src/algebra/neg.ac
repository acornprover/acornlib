/// The "Neg" typeclass represents anything that has a unary negation operator.
typeclass A: Neg {
    neg: A -> A
}
