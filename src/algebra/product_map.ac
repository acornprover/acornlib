from data.basic.set import Set, set_ext, set_preimage, set_preimage_contains_eq,
    preimage_contains, intersection_contains_eq
from data.basic.functions import compose, function_extensionality
from pair import Pair, fan_out, pair_first, pair_second
from analysis import TopologicalSpace, box, box_contains_eq, is_continuous, open_inter,
    continuous_open_preimage
from algebra.product_open_axioms import pair_fst, pair_snd

/// The left projection composed with a fan-out map recovers the first map.
theorem pair_fst_compose_fan_out[Z, X, Y](f: Z -> X, g: Z -> Y) {
    compose(pair_fst[X, Y], fan_out[Z, X, Y](f, g)) = f
} by {
    forall(z: Z) {
        fan_out[Z, X, Y](f, g)(z) = Pair.new(f(z), g(z))
        pair_fst[X, Y](Pair.new(f(z), g(z))) = f(z)
        compose(pair_fst[X, Y], fan_out[Z, X, Y](f, g))(z) = f(z)
    }
    function_extensionality(compose(pair_fst[X, Y], fan_out[Z, X, Y](f, g)), f)
}

/// The right projection composed with a fan-out map recovers the second map.
theorem pair_snd_compose_fan_out[Z, X, Y](f: Z -> X, g: Z -> Y) {
    compose(pair_snd[X, Y], fan_out[Z, X, Y](f, g)) = g
} by {
    forall(z: Z) {
        fan_out[Z, X, Y](f, g)(z) = Pair.new(f(z), g(z))
        pair_snd[X, Y](Pair.new(f(z), g(z))) = g(z)
        compose(pair_snd[X, Y], fan_out[Z, X, Y](f, g))(z) = g(z)
    }
    function_extensionality(compose(pair_snd[X, Y], fan_out[Z, X, Y](f, g)), g)
}

/// The preimage of a box under a fan-out map is the intersection of the
/// component preimages of the box factors.
theorem fan_out_preimage_box[Z, X, Y](
    f: Z -> X, g: Z -> Y, u: Set[X], v: Set[Y]) {
    set_preimage(fan_out[Z, X, Y](f, g), box(u, v))
        = set_preimage(f, u).intersection(set_preimage(g, v))
} by {
    let h = fan_out[Z, X, Y](f, g)
    let l = set_preimage(f, u)
    let r = set_preimage(g, v)
    forall(z: Z) {
        h(z) = Pair.new(f(z), g(z))
        h(z).first = f(z)
        h(z).second = g(z)
        set_preimage_contains_eq(h, box(u, v), z)
        preimage_contains(h, box(u, v), z) = box(u, v).contains(h(z))
        box_contains_eq(u, v, h(z))
        box(u, v).contains(h(z)) = (u.contains(h(z).first) and v.contains(h(z).second))
        box(u, v).contains(h(z)) = (u.contains(f(z)) and v.contains(g(z)))
        set_preimage_contains_eq(f, u, z)
        preimage_contains(f, u, z) = u.contains(f(z))
        set_preimage_contains_eq(g, v, z)
        preimage_contains(g, v, z) = v.contains(g(z))
        intersection_contains_eq(l, r, z)
        l.intersection(r).contains(z) = (u.contains(f(z)) and v.contains(g(z)))
        set_preimage(h, box(u, v)).contains(z) = l.intersection(r).contains(z)
    }
    set_ext(set_preimage(h, box(u, v)), l.intersection(r))
}

/// The preimage of an open box under a fan-out of continuous maps is open.
theorem fan_out_preimage_box_open[Z: TopologicalSpace, X: TopologicalSpace, Y: TopologicalSpace](
    f: Z -> X, g: Z -> Y, u: Set[X], v: Set[Y]) {
    is_continuous(f) and is_continuous(g) and X.is_open(u) and Y.is_open(v)
        implies Z.is_open(set_preimage(fan_out[Z, X, Y](f, g), box(u, v)))
} by {
    if is_continuous(f) and is_continuous(g) and X.is_open(u) and Y.is_open(v) {
        continuous_open_preimage(f, u)
        Z.is_open(set_preimage(f, u))
        continuous_open_preimage(g, v)
        Z.is_open(set_preimage(g, v))
        open_inter(set_preimage(f, u), set_preimage(g, v))
        Z.is_open(set_preimage(f, u).intersection(set_preimage(g, v)))
        fan_out_preimage_box[Z, X, Y](f, g, u, v)
        set_preimage(fan_out[Z, X, Y](f, g), box(u, v)) = set_preimage(f, u).intersection(set_preimage(g, v))
        Z.is_open(set_preimage(fan_out[Z, X, Y](f, g), box(u, v)))
    }
}

