from algebra.mul import Mul
from algebra.one import One
from algebra.semigroup import Semigroup
from algebra.monoid.monoid import Monoid
from algebra.group import Group
from algebra.opposite_algebra import opposite_mul, opposite_mul_eq, opposite_mul_assoc,
    opposite_mul_one_right, opposite_mul_one_left, opposite_mul_inverse_right

/// Carrier wrapper for the multiplicative opposite of a type.
structure MulOpposite[M] {
    /// The underlying value in the original carrier.
    val: M
}

/// Convert an element to the multiplicative-opposite carrier.
define to_mul_opposite[M](x: M) -> MulOpposite[M] {
    MulOpposite.new(x)
}

/// Forget the multiplicative-opposite wrapper.
define of_mul_opposite[M](x: MulOpposite[M]) -> M {
    x.val
}

/// Multiplicative-opposite extensionality from equality of underlying values.
theorem mul_opposite_ext[M](x: MulOpposite[M], y: MulOpposite[M]) {
    x.val = y.val implies x = y
} by {
    if x.val = y.val {
        x = y
    }
}

/// Unwrapping a freshly wrapped element returns the original value.
theorem of_to_mul_opposite[M](x: M) {
    of_mul_opposite(to_mul_opposite(x)) = x
} by {
}

/// Re-wrapping an unwrapped multiplicative-opposite element returns the original wrapper.
theorem to_of_mul_opposite[M](x: MulOpposite[M]) {
    to_mul_opposite(of_mul_opposite(x)) = x
} by {
}

/// The underlying value of `to_mul_opposite` is the original value.
theorem to_mul_opposite_val[M](x: M) {
    to_mul_opposite(x).val = x
} by {
}

attributes MulOpposite[M: Mul] {
    /// Multiplication in the opposite carrier is original multiplication with reversed factors.
    define mul(self, other: MulOpposite[M]) -> MulOpposite[M] {
        MulOpposite.new(opposite_mul(self.val, other.val))
    }
}

/// The multiplicative-opposite wrapper carries reversed multiplication.
instance MulOpposite[M: Mul]: Mul {
    let mul = MulOpposite[M].mul
}

/// The underlying value of a product in the opposite carrier is the reversed original product.
theorem mul_opposite_val_mul[M: Mul](x: MulOpposite[M], y: MulOpposite[M]) {
    (x * y).val = y.val * x.val
} by {
    opposite_mul_eq(x.val, y.val)
}

/// Products of wrapped values unwrap to the original product in reversed order.
theorem mul_opposite_to_mul[M: Mul](x: M, y: M) {
    (to_mul_opposite(x) * to_mul_opposite(y)).val = y * x
} by {
    mul_opposite_val_mul(to_mul_opposite(x), to_mul_opposite(y))
}

/// Wrapping a raw opposite product is the bundled opposite product of wrapped inputs.
theorem to_mul_opposite_opposite_mul[M: Mul](x: M, y: M) {
    to_mul_opposite(opposite_mul(x, y)) = to_mul_opposite(x) * to_mul_opposite(y)
} by {
    to_mul_opposite(opposite_mul(x, y)).val = opposite_mul(x, y)
    opposite_mul_eq(x, y)
    opposite_mul(x, y) = y * x
    mul_opposite_to_mul(x, y)
    (to_mul_opposite(x) * to_mul_opposite(y)).val = y * x
    to_mul_opposite(opposite_mul(x, y)).val = (to_mul_opposite(x) * to_mul_opposite(y)).val
    mul_opposite_ext(to_mul_opposite(opposite_mul(x, y)), to_mul_opposite(x) * to_mul_opposite(y))
}

/// The identity element of the opposite carrier is the original identity.
let mul_opposite_one[M: One]: MulOpposite[M] = MulOpposite.new(M.1)

attributes MulOpposite[M: One] {
    /// The identity element of the opposite carrier is the original identity.
    let one: MulOpposite[M] = mul_opposite_one[M]
}

/// The multiplicative-opposite wrapper carries the original identity element.
instance MulOpposite[M: One]: One {
    let 1: MulOpposite[M] = mul_opposite_one[M]
}

/// The underlying value of the opposite identity is the original identity.
theorem mul_opposite_val_one[M: One] {
    MulOpposite[M].one.val = M.1
} by {
}

/// Opposite multiplication is associative when the original multiplication is associative.
theorem mul_opposite_mul_assoc[M: Semigroup](x: MulOpposite[M], y: MulOpposite[M], z: MulOpposite[M]) {
    x * (y * z) = (x * y) * z
} by {
    mul_opposite_val_mul(y, z)
    mul_opposite_val_mul(x, y * z)

    mul_opposite_val_mul(x, y)
    mul_opposite_val_mul(x * y, z)

    opposite_mul_assoc(x.val, y.val, z.val)
    mul_opposite_ext(x * (y * z), (x * y) * z)
}

/// Opposite-carrier multiplication is associative as a typeclass obligation.
theorem mul_opposite_mul_is_associative[M: Semigroup] {
    forall(x: MulOpposite[M], y: MulOpposite[M], z: MulOpposite[M]) {
        x * (y * z) = (x * y) * z
    }
} by {
    forall(x: MulOpposite[M], y: MulOpposite[M], z: MulOpposite[M]) {
        mul_opposite_mul_assoc(x, y, z)
    }
}

/// Opposite-carrier multiplication is associative in primitive `Mul.mul` form.
theorem mul_opposite_mul_is_associative_raw[M: Semigroup] {
    forall(x: MulOpposite[M], y: MulOpposite[M], z: MulOpposite[M]) {
        Mul.mul(x, Mul.mul(y, z)) = Mul.mul(Mul.mul(x, y), z)
    }
} by {
    forall(x: MulOpposite[M], y: MulOpposite[M], z: MulOpposite[M]) {
        mul_opposite_mul_assoc(x, y, z)
        Mul.mul(x, Mul.mul(y, z)) = Mul.mul(Mul.mul(x, y), z)
    }
}

/// The multiplicative opposite of a semigroup is a semigroup.
instance MulOpposite[M: Semigroup]: Semigroup

/// The original identity is a right identity for opposite-carrier multiplication.
theorem mul_opposite_one_right[M: Monoid](x: MulOpposite[M]) {
    x * MulOpposite[M].one = x
} by {
    mul_opposite_val_mul(x, MulOpposite[M].one)
    mul_opposite_val_one[M]
    opposite_mul_one_right(x.val)
    mul_opposite_ext(x * MulOpposite[M].one, x)
}

/// The original identity is a right identity in primitive `Mul.mul`/`One.1` form.
theorem mul_opposite_one_right_raw[M: Monoid] {
    forall(x: MulOpposite[M]) {
        Mul.mul(x, One.1[MulOpposite[M]]) = x
    }
} by {
    forall(x: MulOpposite[M]) {
        mul_opposite_one_right(x)
        Mul.mul(x, One.1[MulOpposite[M]]) = x
    }
}

/// The original identity is a left identity for opposite-carrier multiplication.
theorem mul_opposite_one_left[M: Monoid](x: MulOpposite[M]) {
    MulOpposite[M].one * x = x
} by {
    mul_opposite_val_mul(MulOpposite[M].one, x)
    mul_opposite_val_one[M]
    opposite_mul_one_left(x.val)
    mul_opposite_ext(MulOpposite[M].one * x, x)
}

/// The original identity is a left identity in primitive `Mul.mul`/`One.1` form.
theorem mul_opposite_one_left_raw[M: Monoid] {
    forall(x: MulOpposite[M]) {
        Mul.mul(One.1[MulOpposite[M]], x) = x
    }
} by {
    forall(x: MulOpposite[M]) {
        mul_opposite_one_left(x)
        Mul.mul(One.1[MulOpposite[M]], x) = x
    }
}

/// The multiplicative opposite of a monoid is a monoid.
instance MulOpposite[M: Monoid]: Monoid

attributes MulOpposite[G: Group] {
    /// Inversion in the opposite carrier is inherited from the original carrier.
    define inverse(self) -> MulOpposite[G] {
        MulOpposite.new(self.val.inverse)
    }
}

/// The underlying value of an opposite inverse is the original inverse.
theorem mul_opposite_val_inverse[G: Group](x: MulOpposite[G]) {
    x.inverse.val = x.val.inverse
} by {
}

/// The inherited inverse is a right inverse for opposite-carrier multiplication.
theorem mul_opposite_inverse_right[G: Group](x: MulOpposite[G]) {
    x * x.inverse = MulOpposite[G].one
} by {
    mul_opposite_val_mul(x, x.inverse)
    mul_opposite_val_inverse(x)
    opposite_mul_inverse_right(x.val)
    mul_opposite_val_one[G]
    mul_opposite_ext(x * x.inverse, MulOpposite[G].one)
}

/// The inherited inverse is a right inverse in primitive `Mul.mul`/`One.1` form.
theorem mul_opposite_inverse_right_raw[G: Group] {
    forall(x: MulOpposite[G]) {
        Mul.mul(x, x.inverse) = One.1[MulOpposite[G]]
    }
} by {
    forall(x: MulOpposite[G]) {
        One.1[MulOpposite[G]] = MulOpposite[G].one
        mul_opposite_inverse_right(x)
        Mul.mul(x, x.inverse) = One.1[MulOpposite[G]]
    }
}

/// The multiplicative opposite of a group is a group.
instance MulOpposite[G: Group]: Group {
    let inverse = MulOpposite[G].inverse
}
