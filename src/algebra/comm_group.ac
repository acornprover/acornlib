from algebra.comm_monoid import CommMonoid
from algebra.group import Group

/// CommGroup represents an Abelian group. It's a commutative, multiplicative group.
typeclass C: CommGroup extends CommMonoid, Group {
    // All the properties are provided by the base typeclasses.
}