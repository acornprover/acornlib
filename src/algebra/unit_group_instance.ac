/// Typeclass instances for the group of units of a monoid.

from algebra.mul import Mul
from algebra.one import One
from algebra.semigroup import Semigroup
from algebra.monoid.monoid import Monoid
from algebra.units import Unit, unit_one, unit_mul, unit_inverse, unit_mul_val,
    unit_inverse_val, unit_inverse_inv, unit_mul_inv, unit_mul_one_right, unit_mul_one_left,
    unit_mul_inverse_right, unit_mul_assoc, unit_ext_val, unit_eq_iff_val_eq,
    unit_eq_one_iff_val_eq_one

/// Unit multiplication is the existing bundled product of units.
instance Unit[M: Monoid]: Mul {
    define mul(self, other: Unit[M]) -> Unit[M] {
        unit_mul(self, other)
    }
}

/// The identity unit is the monoid identity for the typeclass structure.
instance Unit[M: Monoid]: One {
    let 1: Unit[M] = unit_one[M]
}

/// The typeclass product agrees with `unit_mul`.
theorem unit_typeclass_mul_eq_unit_mul[M: Monoid](u: Unit[M], v: Unit[M]) {
    u * v = unit_mul(u, v)
}

/// The typeclass identity agrees with `unit_one`.
theorem unit_typeclass_one_eq_unit_one[M: Monoid] {
    One.1[Unit[M]] = unit_one[M]
}

/// Unit multiplication is associative in typeclass notation.
theorem unit_typeclass_mul_assoc[M: Monoid](u: Unit[M], v: Unit[M], w: Unit[M]) {
    Mul.mul(u, Mul.mul(v, w)) = Mul.mul(Mul.mul(u, v), w)
} by {
    v * w = unit_mul(v, w)
    u * (v * w) = unit_mul(u, unit_mul(v, w))
    u * v = unit_mul(u, v)
    (u * v) * w = unit_mul(unit_mul(u, v), w)
    unit_mul_assoc(u, v, w)
    unit_mul(u, unit_mul(v, w)) = unit_mul(unit_mul(u, v), w)
    u * (v * w) = (u * v) * w
}

/// Unit multiplication satisfies the semigroup law in universal form.
theorem unit_semigroup_mul_associative[M: Monoid] {
    forall(u: Unit[M], v: Unit[M], w: Unit[M]) {
        Mul.mul(u, Mul.mul(v, w)) = Mul.mul(Mul.mul(u, v), w)
    }
} by {
    if not forall(u: Unit[M], v: Unit[M], w: Unit[M]) {
        Mul.mul(u, Mul.mul(v, w)) = Mul.mul(Mul.mul(u, v), w)
    } {
        let (u0: Unit[M], v0: Unit[M], w0: Unit[M]) satisfy {
            Mul.mul(u0, Mul.mul(v0, w0)) != Mul.mul(Mul.mul(u0, v0), w0)
        }
        unit_typeclass_mul_assoc(u0, v0, w0)
        false
    }
}

/// Units form a semigroup under typeclass multiplication.
instance Unit[M: Monoid]: Semigroup

/// Multiplying a unit by the typeclass identity on the right changes nothing.
theorem unit_typeclass_mul_one_right[M: Monoid](u: Unit[M]) {
    u * One.1[Unit[M]] = u
} by {
    One.1[Unit[M]] = unit_one[M]
    u * One.1[Unit[M]] = unit_mul(u, unit_one[M])
    unit_mul_one_right(u)
    unit_mul(u, unit_one[M]) = u
    u * One.1[Unit[M]] = u
}

/// Multiplying a unit by the typeclass identity on the left changes nothing.
theorem unit_typeclass_mul_one_left[M: Monoid](u: Unit[M]) {
    One.1[Unit[M]] * u = u
} by {
    One.1[Unit[M]] = unit_one[M]
    One.1[Unit[M]] * u = unit_mul(unit_one[M], u)
    unit_mul_one_left(u)
    unit_mul(unit_one[M], u) = u
    One.1[Unit[M]] * u = u
}

/// The unit identity satisfies the right monoid identity law in universal form.
theorem unit_monoid_identity_right[M: Monoid] {
    forall(u: Unit[M]) {
        u * One.1[Unit[M]] = u
    }
} by {
    forall(u: Unit[M]) {
        unit_typeclass_mul_one_right(u)
    }
}

/// The unit identity satisfies the left monoid identity law in universal form.
theorem unit_monoid_identity_left[M: Monoid] {
    forall(u: Unit[M]) {
        One.1[Unit[M]] * u = u
    }
} by {
    forall(u: Unit[M]) {
        unit_typeclass_mul_one_left(u)
    }
}

/// Units form a monoid under typeclass multiplication and identity.
instance Unit[M: Monoid]: Monoid

/// Multiplying a unit by the bundled inverse on the right gives the typeclass identity.
theorem unit_typeclass_unit_inverse_right[M: Monoid](u: Unit[M]) {
    u * unit_inverse(u) = One.1[Unit[M]]
} by {
    One.1[Unit[M]] = unit_one[M]
    u * unit_inverse(u) = unit_mul(u, unit_inverse(u))
    unit_mul_inverse_right(u)
    unit_mul(u, unit_inverse(u)) = unit_one[M]
    u * unit_inverse(u) = One.1[Unit[M]]
}

from algebra.group import Group

/// Units of a monoid form a group.
instance Unit[M: Monoid]: Group {
    define inverse(self) -> Unit[M] {
        unit_inverse(self)
    }
}

/// The typeclass inverse agrees with `unit_inverse`.
theorem unit_typeclass_inverse_eq_unit_inverse[M: Monoid](u: Unit[M]) {
    u.inverse = unit_inverse(u)
}

/// Multiplying a unit by its typeclass inverse on the right gives the typeclass identity.
theorem unit_typeclass_inverse_right[M: Monoid](u: Unit[M]) {
    u * u.inverse = One.1[Unit[M]]
} by {
    u.inverse = unit_inverse(u)
    unit_typeclass_unit_inverse_right(u)
    u * unit_inverse(u) = One.1[Unit[M]]
    u * u.inverse = One.1[Unit[M]]
}

/// The value of a typeclass product is the product of values.
theorem unit_val_mul_typeclass[M: Monoid](u: Unit[M], v: Unit[M]) {
    (u * v).val = u.val * v.val
} by {
    u * v = unit_mul(u, v)
    unit_mul_val(u, v)
    (u * v).val = u.val * v.val
}

/// The inverse component of a typeclass product is the reverse product of inverse components.
theorem unit_inv_mul_typeclass[M: Monoid](u: Unit[M], v: Unit[M]) {
    (u * v).inv = v.inv * u.inv
} by {
    u * v = unit_mul(u, v)
    unit_mul_inv(u, v)
    (u * v).inv = v.inv * u.inv
}

/// The typeclass identity unit has value equal to the monoid identity.
theorem unit_one_val_typeclass[M: Monoid] {
    One.1[Unit[M]].val = M.1
} by {
    One.1[Unit[M]] = unit_one[M]
    unit_one[M].val = M.1
    One.1[Unit[M]].val = M.1
}

/// The typeclass identity unit has inverse component equal to the monoid identity.
theorem unit_one_inv_typeclass[M: Monoid] {
    One.1[Unit[M]].inv = M.1
} by {
    One.1[Unit[M]] = unit_one[M]
    unit_one[M].inv = M.1
    One.1[Unit[M]].inv = M.1
}

/// The value of the typeclass inverse is the stored inverse of the unit.
theorem unit_inverse_val_typeclass[M: Monoid](u: Unit[M]) {
    u.inverse.val = u.inv
} by {
    u.inverse = unit_inverse(u)
    unit_inverse_val(u)
    u.inverse.val = u.inv
}

/// The inverse component of the typeclass inverse is the value of the unit.
theorem unit_inverse_inv_typeclass[M: Monoid](u: Unit[M]) {
    u.inverse.inv = u.val
} by {
    u.inverse = unit_inverse(u)
    unit_inverse_inv(u)
    u.inverse.inv = u.val
}

/// Equality of units can be checked after projecting values, in typeclass notation.
theorem unit_typeclass_eq_iff_val_eq[M: Monoid](u: Unit[M], v: Unit[M]) {
    (u = v) = (u.val = v.val)
} by {
    unit_eq_iff_val_eq(u, v)
}

/// A typeclass unit is the identity exactly when its value is the monoid identity.
theorem unit_typeclass_eq_one_iff_val_eq_one[M: Monoid](u: Unit[M]) {
    (u = One.1[Unit[M]]) = (u.val = M.1)
} by {
    One.1[Unit[M]] = unit_one[M]
    unit_eq_one_iff_val_eq_one(u)
}

/// The typeclass product is determined by the product of projected values.
theorem unit_typeclass_mul_ext_val[M: Monoid](u: Unit[M], v: Unit[M], w: Unit[M]) {
    u.val * v.val = w.val implies u * v = w
} by {
    if u.val * v.val = w.val {
        unit_val_mul_typeclass(u, v)
        (u * v).val = w.val
        unit_ext_val(u * v, w)
        u * v = w
    }
}
