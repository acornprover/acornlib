from algebra.ring.ring import Ring
from algebra.ring.ring_hom import RingHom, ring_hom_zero, ring_hom_one, ring_hom_add, ring_hom_neg, ring_hom_mul,
    identity_ring_hom, identity_ring_hom_hom, compose_ring_hom, compose_ring_hom_hom
from algebra.add_subgroup import AddSubgroup, add_zero_constraint, add_closure_constraint, neg_constraint,
    add_subgroup_constraint
from algebra.monoid.submonoid import Submonoid, submonoid_identity_constraint, submonoid_closure_constraint,
    submonoid_constraint
from data.basic.set import Set, singleton_contains_eq, singleton_set_is_finite, is_subset_monotone_map,
    is_set_extensive_map, is_set_idempotent_map, is_set_closure_operator,
    union_contains_eq, union_contains_left, union_contains_right
from data.basic.functions import identity_fn, predicate_extensionality

/// True if a subset contains the additive identity.
define subring_zero_constraint[R: Ring](contains: R -> Bool) -> Bool {
    contains(R.0)
}

/// True if a subset contains the multiplicative identity.
define subring_one_constraint[R: Ring](contains: R -> Bool) -> Bool {
    contains(R.1)
}

/// True if a subset is closed under addition.
define subring_add_constraint[R: Ring](contains: R -> Bool) -> Bool {
    forall(a: R, b: R) {
        contains(a) and contains(b) implies contains(a + b)
    }
}

/// True if a subset is closed under additive negation.
define subring_neg_constraint[R: Ring](contains: R -> Bool) -> Bool {
    forall(a: R) {
        contains(a) implies contains(-a)
    }
}

/// True if a subset is closed under multiplication.
define subring_mul_constraint[R: Ring](contains: R -> Bool) -> Bool {
    forall(a: R, b: R) {
        contains(a) and contains(b) implies contains(a * b)
    }
}

/// True if a subset satisfies all the requirements to be a subring.
define subring_constraint[R: Ring](contains: R -> Bool) -> Bool {
    subring_zero_constraint(contains) and subring_one_constraint(contains) and
        subring_add_constraint(contains) and subring_neg_constraint(contains) and
        subring_mul_constraint(contains)
}

/// A subring of a ring, represented as a subset containing zero and one and closed under ring operations.
structure Subring[R: Ring] {
    /// True if the given element is a member of this subring.
    contains: R -> Bool
} constraint {
    subring_constraint(contains)
}

/// Subring extensionality from equality of membership.
theorem subring_ext[R: Ring](a: Subring[R], b: Subring[R]) {
    (forall(x: R) { a.contains(x) = b.contains(x) }) implies a = b
} by {
    if forall(x: R) { a.contains(x) = b.contains(x) } {
        a.contains = b.contains
    }
}

/// Equal subrings have equal membership predicates.
theorem subring_eq_contains[R: Ring](a: Subring[R], b: Subring[R]) {
    a = b implies a.contains = b.contains
}

/// Equal subrings have equal membership at every element.
theorem subring_eq_contains_at[R: Ring](a: Subring[R], b: Subring[R], x: R) {
    a = b implies a.contains(x) = b.contains(x)
} by {
    if a = b {
        a.contains = b.contains
        a.contains(x) = b.contains(x)
    }
}

/// Equality of subrings transports predicates on subrings.
theorem subring_eq_transport_predicate[R: Ring](
    p: Subring[R] -> Bool,
    a: Subring[R],
    b: Subring[R]
) {
    a = b and p(a) implies p(b)
}

/// Equality of subrings transports predicates on subrings in the reverse direction.
theorem subring_eq_transport_predicate_rev[R: Ring](
    p: Subring[R] -> Bool,
    a: Subring[R],
    b: Subring[R]
) {
    a = b and p(b) implies p(a)
}

/// Equality of membership predicates determines equality of subrings.
theorem subring_eq_of_contains_eq[R: Ring](a: Subring[R], b: Subring[R]) {
    a.contains = b.contains implies a = b
} by {
    if a.contains = b.contains {
        forall(x: R) {
            a.contains(x) = b.contains(x)
        }
        subring_ext(a, b)
    }
}

/// Pointwise equality of membership determines equality of subrings.
theorem subring_eq_of_contains_at_eq[R: Ring](a: Subring[R], b: Subring[R]) {
    (forall(x: R) { a.contains(x) = b.contains(x) }) implies a = b
} by {
    if forall(x: R) { a.contains(x) = b.contains(x) } {
        subring_ext(a, b)
    }
}

/// Membership of zero in any subring.
theorem subring_contains_zero[R: Ring](s: Subring[R]) {
    s.contains(R.0)
} by {
    subring_constraint(s.contains)
    subring_constraint(s.contains) =
        (subring_zero_constraint(s.contains) and subring_one_constraint(s.contains) and
            subring_add_constraint(s.contains) and subring_neg_constraint(s.contains) and
            subring_mul_constraint(s.contains))
    subring_zero_constraint(s.contains)
    subring_zero_constraint(s.contains) = s.contains(R.0)
}

/// Membership of one in any subring.
theorem subring_contains_one[R: Ring](s: Subring[R]) {
    s.contains(R.1)
} by {
    subring_constraint(s.contains)
    subring_constraint(s.contains) =
        (subring_zero_constraint(s.contains) and subring_one_constraint(s.contains) and
            subring_add_constraint(s.contains) and subring_neg_constraint(s.contains) and
            subring_mul_constraint(s.contains))
    subring_one_constraint(s.contains)
    subring_one_constraint(s.contains) = s.contains(R.1)
}

/// Closure of any subring under addition.
theorem subring_contains_add[R: Ring](s: Subring[R], a: R, b: R) {
    s.contains(a) and s.contains(b) implies s.contains(a + b)
} by {
    if s.contains(a) and s.contains(b) {
        subring_constraint(s.contains)
        subring_constraint(s.contains) =
            (subring_zero_constraint(s.contains) and subring_one_constraint(s.contains) and
                subring_add_constraint(s.contains) and subring_neg_constraint(s.contains) and
                subring_mul_constraint(s.contains))
        subring_add_constraint(s.contains)
        subring_add_constraint(s.contains) = forall(x: R, y: R) {
            s.contains(x) and s.contains(y) implies s.contains(x + y)
        }
        s.contains(a + b)
    }
}

/// Closure of any subring under additive negation.
theorem subring_contains_neg[R: Ring](s: Subring[R], a: R) {
    s.contains(a) implies s.contains(-a)
} by {
    if s.contains(a) {
        subring_constraint(s.contains)
        subring_constraint(s.contains) =
            (subring_zero_constraint(s.contains) and subring_one_constraint(s.contains) and
                subring_add_constraint(s.contains) and subring_neg_constraint(s.contains) and
                subring_mul_constraint(s.contains))
        subring_neg_constraint(s.contains)
        subring_neg_constraint(s.contains) = forall(x: R) {
            s.contains(x) implies s.contains(-x)
        }
        s.contains(-a)
    }
}

/// Closure of any subring under subtraction.
theorem subring_contains_sub[R: Ring](s: Subring[R], a: R, b: R) {
    s.contains(a) and s.contains(b) implies s.contains(a - b)
} by {
    if s.contains(a) and s.contains(b) {
        subring_contains_neg(s, b)
        s.contains(-b)
        subring_contains_add(s, a, -b)
        s.contains(a + -b)
        a - b = a + -b
    }
}

/// Closure of any subring under multiplication.
theorem subring_contains_mul[R: Ring](s: Subring[R], a: R, b: R) {
    s.contains(a) and s.contains(b) implies s.contains(a * b)
} by {
    if s.contains(a) and s.contains(b) {
        subring_constraint(s.contains)
        subring_constraint(s.contains) =
            (subring_zero_constraint(s.contains) and subring_one_constraint(s.contains) and
                subring_add_constraint(s.contains) and subring_neg_constraint(s.contains) and
                subring_mul_constraint(s.contains))
        subring_mul_constraint(s.contains)
        subring_mul_constraint(s.contains) = forall(x: R, y: R) {
            s.contains(x) and s.contains(y) implies s.contains(x * y)
        }
        s.contains(a * b)
    }
}

/// The common membership predicate of two subrings.
define subring_intersection_contains[R: Ring](a: Subring[R], b: Subring[R], x: R) -> Bool {
    a.contains(x) and b.contains(x)
}

/// The common part of two subrings contains zero.
theorem subring_intersection_zero_constraint[R: Ring](a: Subring[R], b: Subring[R]) {
    subring_zero_constraint(subring_intersection_contains(a, b))
} by {
    subring_contains_zero(a)
    subring_contains_zero(b)
    subring_intersection_contains(a, b, R.0)
}

/// The common part of two subrings contains one.
theorem subring_intersection_one_constraint[R: Ring](a: Subring[R], b: Subring[R]) {
    subring_one_constraint(subring_intersection_contains(a, b))
} by {
    subring_contains_one(a)
    subring_contains_one(b)
    subring_intersection_contains(a, b, R.1)
}

/// The common part of two subrings is closed under addition.
theorem subring_intersection_add_constraint[R: Ring](a: Subring[R], b: Subring[R]) {
    subring_add_constraint(subring_intersection_contains(a, b))
} by {
    forall(x: R, y: R) {
        if subring_intersection_contains(a, b, x) and subring_intersection_contains(a, b, y) {
            a.contains(x)
            b.contains(x)
            a.contains(y)
            b.contains(y)
            subring_contains_add(a, x, y)
            subring_contains_add(b, x, y)
            a.contains(x + y)
            b.contains(x + y)
            subring_intersection_contains(a, b, x + y)
        }
    }
}

/// The common part of two subrings is closed under additive negation.
theorem subring_intersection_neg_constraint[R: Ring](a: Subring[R], b: Subring[R]) {
    subring_neg_constraint(subring_intersection_contains(a, b))
} by {
    forall(x: R) {
        if subring_intersection_contains(a, b, x) {
            a.contains(x)
            b.contains(x)
            subring_contains_neg(a, x)
            subring_contains_neg(b, x)
            a.contains(-x)
            b.contains(-x)
            subring_intersection_contains(a, b, -x)
        }
    }
}

/// The common part of two subrings is closed under multiplication.
theorem subring_intersection_mul_constraint[R: Ring](a: Subring[R], b: Subring[R]) {
    subring_mul_constraint(subring_intersection_contains(a, b))
} by {
    forall(x: R, y: R) {
        if subring_intersection_contains(a, b, x) and subring_intersection_contains(a, b, y) {
            a.contains(x)
            b.contains(x)
            a.contains(y)
            b.contains(y)
            subring_contains_mul(a, x, y)
            subring_contains_mul(b, x, y)
            a.contains(x * y)
            b.contains(x * y)
            subring_intersection_contains(a, b, x * y)
        }
    }
}

/// The common part of two subrings is a subring.
theorem subring_intersection_constraint[R: Ring](a: Subring[R], b: Subring[R]) {
    subring_constraint(subring_intersection_contains(a, b))
} by {
    subring_intersection_zero_constraint(a, b)
    subring_intersection_one_constraint(a, b)
    subring_intersection_add_constraint(a, b)
    subring_intersection_neg_constraint(a, b)
    subring_intersection_mul_constraint(a, b)
}

/// The common part of two subrings.
let subring_intersection[R: Ring](a: Subring[R], b: Subring[R]) -> result: Subring[R] satisfy {
    Subring.new(subring_intersection_contains(a, b)) = Option.some(result)
} by {
    subring_intersection_constraint(a, b)
}

attributes Subring[R: Ring] {
    /// Subring extensionality from pointwise equality of membership.
    let ext = subring_ext[R]

    /// The subset of ring elements belonging to this subring.
    define as_set(self) -> Set[R] {
        Set[R].new(self.contains)
    }

    /// The common part of two subrings.
    let intersection: (Subring[R], Subring[R]) -> Subring[R] = subring_intersection
}

/// Membership in the underlying set is membership in the subring.
theorem subring_as_set_contains_eq[R: Ring](s: Subring[R], x: R) {
    s.as_set.contains(x) = s.contains(x)
} by {
    s.as_set.contains(x) = s.contains(x)
}

/// Membership in a subring is membership in its underlying set.
theorem subring_contains_as_set_eq[R: Ring](s: Subring[R], x: R) {
    s.contains(x) = s.as_set.contains(x)
} by {
    subring_as_set_contains_eq(s, x)
    s.contains(x) = s.as_set.contains(x)
}

/// Equal subrings have equal underlying sets.
theorem subring_eq_as_set[R: Ring](a: Subring[R], b: Subring[R]) {
    a = b implies a.as_set = b.as_set
}

/// Membership in the intersection means membership in both subrings.
theorem subring_intersection_contains_eq[R: Ring](a: Subring[R], b: Subring[R], x: R) {
    a.intersection(b).contains(x) = (a.contains(x) and b.contains(x))
} by {
    a.intersection(b).contains(x) = subring_intersection_contains(a, b, x)
    subring_intersection_contains(a, b, x) = (a.contains(x) and b.contains(x))
}

/// The intersection is contained in the left subring.
theorem subring_intersection_subset_left[R: Ring](a: Subring[R], b: Subring[R], x: R) {
    a.intersection(b).contains(x) implies a.contains(x)
} by {
    if a.intersection(b).contains(x) {
        subring_intersection_contains_eq(a, b, x)
        a.contains(x)
    }
}

/// The intersection is contained in the right subring.
theorem subring_intersection_subset_right[R: Ring](a: Subring[R], b: Subring[R], x: R) {
    a.intersection(b).contains(x) implies b.contains(x)
} by {
    if a.intersection(b).contains(x) {
        subring_intersection_contains_eq(a, b, x)
        b.contains(x)
    }
}

/// Subring intersection is commutative.
theorem subring_intersection_comm[R: Ring](a: Subring[R], b: Subring[R]) {
    a.intersection(b) = b.intersection(a)
} by {
    forall(x: R) {
        if a.intersection(b).contains(x) {
            subring_intersection_contains_eq(a, b, x)
            a.contains(x)
            b.contains(x)
            subring_intersection_contains_eq(b, a, x)
            b.intersection(a).contains(x)
        }
        if b.intersection(a).contains(x) {
            subring_intersection_contains_eq(b, a, x)
            b.contains(x)
            a.contains(x)
            subring_intersection_contains_eq(a, b, x)
            a.intersection(b).contains(x)
        }
        a.intersection(b).contains(x) = b.intersection(a).contains(x)
    }
    subring_ext(a.intersection(b), b.intersection(a))
}

/// Subring intersection is associative.
theorem subring_intersection_assoc[R: Ring](a: Subring[R], b: Subring[R], c: Subring[R]) {
    a.intersection(b).intersection(c) = a.intersection(b.intersection(c))
} by {
    let lhs = a.intersection(b).intersection(c)
    let rhs = a.intersection(b.intersection(c))
    forall(x: R) {
        if lhs.contains(x) {
            subring_intersection_contains_eq(a.intersection(b), c, x)
            a.intersection(b).contains(x)
            c.contains(x)
            subring_intersection_contains_eq(a, b, x)
            a.contains(x)
            b.contains(x)
            subring_intersection_contains_eq(b, c, x)
            b.intersection(c).contains(x)
            subring_intersection_contains_eq(a, b.intersection(c), x)
            rhs.contains(x)
        }
        if rhs.contains(x) {
            subring_intersection_contains_eq(a, b.intersection(c), x)
            a.contains(x)
            b.intersection(c).contains(x)
            subring_intersection_contains_eq(b, c, x)
            b.contains(x)
            c.contains(x)
            subring_intersection_contains_eq(a, b, x)
            a.intersection(b).contains(x)
            subring_intersection_contains_eq(a.intersection(b), c, x)
            lhs.contains(x)
        }
        lhs.contains(x) = rhs.contains(x)
    }
    subring_ext(lhs, rhs)
}

/// Subring intersection is idempotent.
theorem subring_intersection_idempotent[R: Ring](a: Subring[R]) {
    a.intersection(a) = a
} by {
    forall(x: R) {
        if a.intersection(a).contains(x) {
            subring_intersection_contains_eq(a, a, x)
            a.contains(x)
        }
        if a.contains(x) {
            subring_intersection_contains_eq(a, a, x)
            a.intersection(a).contains(x)
        }
        a.intersection(a).contains(x) = a.contains(x)
    }
    subring_ext(a.intersection(a), a)
}

/// True if every element of one subring belongs to another.
define subring_subset[R: Ring](a: Subring[R], b: Subring[R]) -> Bool {
    forall(x: R) {
        a.contains(x) implies b.contains(x)
    }
}

/// Subring containment is containment of the underlying sets.
theorem subring_subset_as_set_eq[R: Ring](a: Subring[R], b: Subring[R]) {
    subring_subset(a, b) = a.as_set.subset(b.as_set)
} by {
    if subring_subset(a, b) {
        forall(x: R) {
            if a.as_set.contains(x) {
                subring_as_set_contains_eq(a, x)
                a.contains(x)
                b.contains(x)
                subring_as_set_contains_eq(b, x)
                b.as_set.contains(x)
            }
        }
        a.as_set.subset(b.as_set)
    }
    if a.as_set.subset(b.as_set) {
        forall(x: R) {
            if a.contains(x) {
                subring_as_set_contains_eq(a, x)
                a.as_set.contains(x)
                a.as_set.subset(b.as_set) = forall(y: R) {
                    a.as_set.contains(y) implies b.as_set.contains(y)
                }
                b.as_set.contains(x)
                subring_as_set_contains_eq(b, x)
                b.contains(x)
            }
        }
        subring_subset(a, b)
    }
    subring_subset(a, b) = a.as_set.subset(b.as_set)
}

/// Subring containment gives containment of underlying sets.
theorem subring_as_set_subset_of_subset[R: Ring](a: Subring[R], b: Subring[R]) {
    subring_subset(a, b) implies a.as_set.subset(b.as_set)
} by {
    if subring_subset(a, b) {
        subring_subset_as_set_eq(a, b)
        a.as_set.subset(b.as_set)
    }
}

/// Containment of underlying sets gives subring containment.
theorem subring_subset_of_as_set_subset[R: Ring](a: Subring[R], b: Subring[R]) {
    a.as_set.subset(b.as_set) implies subring_subset(a, b)
} by {
    if a.as_set.subset(b.as_set) {
        subring_subset_as_set_eq(a, b)
        subring_subset(a, b)
    }
}

/// Subring inclusion is reflexive.
theorem subring_subset_refl[R: Ring](a: Subring[R]) {
    subring_subset(a, a)
} by {
    forall(x: R) {
        a.contains(x) implies a.contains(x)
    }
}

/// Subring inclusion is transitive.
theorem subring_subset_trans[R: Ring](a: Subring[R], b: Subring[R], c: Subring[R]) {
    subring_subset(a, b) and subring_subset(b, c) implies subring_subset(a, c)
} by {
    if subring_subset(a, b) and subring_subset(b, c) {
        forall(x: R) {
            if a.contains(x) {
                b.contains(x)
                c.contains(x)
            }
        }
    }
}

/// The intersection of two subrings is contained in the left subring.
theorem subring_intersection_subset_left_relation[R: Ring](a: Subring[R], b: Subring[R]) {
    subring_subset(a.intersection(b), a)
} by {
    forall(x: R) {
        if a.intersection(b).contains(x) {
            subring_intersection_contains_eq(a, b, x)
            a.contains(x)
        }
    }
}

/// The intersection of two subrings is contained in the right subring.
theorem subring_intersection_subset_right_relation[R: Ring](a: Subring[R], b: Subring[R]) {
    subring_subset(a.intersection(b), b)
} by {
    forall(x: R) {
        if a.intersection(b).contains(x) {
            subring_intersection_contains_eq(a, b, x)
            b.contains(x)
        }
    }
}

/// Every common subring of two subrings is contained in their intersection.
theorem subring_subset_intersection_of_subset_left_right[R: Ring](
    c: Subring[R],
    a: Subring[R],
    b: Subring[R]
) {
    subring_subset(c, a) and subring_subset(c, b) implies subring_subset(c, a.intersection(b))
} by {
    if subring_subset(c, a) and subring_subset(c, b) {
        forall(x: R) {
            if c.contains(x) {
                a.contains(x)
                b.contains(x)
                subring_intersection_contains_eq(a, b, x)
                a.intersection(b).contains(x)
            }
        }
    }
}

/// Containment in an intersection is equivalent to containment in both subrings.
theorem subring_subset_intersection_iff[R: Ring](c: Subring[R], a: Subring[R], b: Subring[R]) {
    subring_subset(c, a.intersection(b)) = (subring_subset(c, a) and subring_subset(c, b))
} by {
    if subring_subset(c, a.intersection(b)) {
        subring_intersection_subset_left_relation(a, b)
        subring_subset_trans(c, a.intersection(b), a)
        subring_subset(c, a)
        subring_intersection_subset_right_relation(a, b)
        subring_subset_trans(c, a.intersection(b), b)
        subring_subset(c, b)
        subring_subset(c, a) and subring_subset(c, b)
    }
    if subring_subset(c, a) and subring_subset(c, b) {
        subring_subset_intersection_of_subset_left_right(c, a, b)
        subring_subset(c, a.intersection(b))
    }
    subring_subset(c, a.intersection(b)) = (subring_subset(c, a) and subring_subset(c, b))
}

/// Mutual inclusion of subrings forces equality.
theorem subring_subset_antisymm[R: Ring](a: Subring[R], b: Subring[R]) {
    subring_subset(a, b) and subring_subset(b, a) implies a = b
} by {
    if subring_subset(a, b) and subring_subset(b, a) {
        subring_subset(a, b) = forall(y: R) {
            a.contains(y) implies b.contains(y)
        }
        subring_subset(b, a) = forall(y: R) {
            b.contains(y) implies a.contains(y)
        }
        forall(x: R) {
            if a.contains(x) {
                b.contains(x)
            }
            if b.contains(x) {
                a.contains(x)
            }
            a.contains(x) = b.contains(x)
        }
        predicate_extensionality(a.contains, b.contains)
        a.contains = b.contains
        a = b
    }
}

/// Equality of subrings is equivalent to mutual inclusion.
theorem subring_eq_iff_subset_both[R: Ring](a: Subring[R], b: Subring[R]) {
    a = b = (subring_subset(a, b) and subring_subset(b, a))
} by {
    if a = b {
        subring_subset_refl(a)
        subring_subset(a, b)
        subring_subset(b, a)
        subring_subset(a, b) and subring_subset(b, a)
    }
    if subring_subset(a, b) and subring_subset(b, a) {
        subring_subset_antisymm(a, b)
        a = b
    }
    a = b = (subring_subset(a, b) and subring_subset(b, a))
}

/// Intersecting with a larger subring gives the smaller subring.
theorem subring_intersection_eq_left_of_subset[R: Ring](a: Subring[R], b: Subring[R]) {
    subring_subset(a, b) implies a.intersection(b) = a
} by {
    if subring_subset(a, b) {
        subring_intersection_subset_left_relation(a, b)
        subring_subset_intersection_of_subset_left_right(a, a, b)
        subring_subset(a, a.intersection(b))
        subring_subset_antisymm(a.intersection(b), a)
        a.intersection(b) = a
    }
}

/// Intersecting with a larger subring on the left gives the smaller subring.
theorem subring_intersection_eq_right_of_subset[R: Ring](a: Subring[R], b: Subring[R]) {
    subring_subset(b, a) implies a.intersection(b) = b
} by {
    if subring_subset(b, a) {
        subring_intersection_subset_right_relation(a, b)
        subring_subset_intersection_of_subset_left_right(b, a, b)
        subring_subset(b, a.intersection(b))
        subring_subset_antisymm(a.intersection(b), b)
        a.intersection(b) = b
    }
}

/// The full membership predicate is a subring predicate.
define full_subring_contains[R: Ring](a: R) -> Bool {
    true
}

/// The full subset of any ring is a subring.
theorem full_subring_constraint[R: Ring] {
    subring_constraint(full_subring_contains[R])
} by {
    full_subring_contains[R](R.0)
    full_subring_contains[R](R.1)
    subring_zero_constraint(full_subring_contains[R])
    subring_one_constraint(full_subring_contains[R])
    forall(a: R, b: R) {
        if full_subring_contains[R](a) and full_subring_contains[R](b) {
            full_subring_contains[R](a + b)
            full_subring_contains[R](a * b)
        }
    }
    subring_add_constraint(full_subring_contains[R])
    forall(a: R) {
        if full_subring_contains[R](a) {
            full_subring_contains[R](-a)
        }
    }
    subring_neg_constraint(full_subring_contains[R])
    subring_mul_constraint(full_subring_contains[R])
}

/// The full subring, containing every element.
let full_subring[R: Ring]: Subring[R] satisfy {
    Subring.new(full_subring_contains[R]) = Option.some(full_subring)
}

/// Every element belongs to the full subring.
theorem full_subring_contains_everything[R: Ring](a: R) {
    full_subring[R].contains(a)
}

/// Membership in the full subring is always true.
theorem full_subring_contains_eq[R: Ring](a: R) {
    full_subring[R].contains(a) = true
} by {
    full_subring_contains_everything(a)
}

/// Every subring is contained in the full subring.
theorem subring_subset_full[R: Ring](s: Subring[R]) {
    subring_subset(s, full_subring[R])
} by {
    forall(x: R) {
        if s.contains(x) {
            full_subring_contains_everything(x)
        }
    }
}

/// True if every element of a set belongs to a subring.
define set_subset_subring[R: Ring](a: Set[R], s: Subring[R]) -> Bool {
    forall(x: R) {
        a.contains(x) implies s.contains(x)
    }
}

/// Set containment in a subring is containment in its underlying set.
theorem set_subset_subring_as_set_eq[R: Ring](a: Set[R], s: Subring[R]) {
    set_subset_subring(a, s) = a.subset(s.as_set)
} by {
    if set_subset_subring(a, s) {
        forall(x: R) {
            if a.contains(x) {
                s.contains(x)
                subring_as_set_contains_eq(s, x)
                s.as_set.contains(x)
            }
        }
        a.subset(s.as_set)
    }
    if a.subset(s.as_set) {
        forall(x: R) {
            if a.contains(x) {
                a.subset(s.as_set) = forall(y: R) {
                    a.contains(y) implies s.as_set.contains(y)
                }
                s.as_set.contains(x)
                subring_as_set_contains_eq(s, x)
                s.contains(x)
            }
        }
        set_subset_subring(a, s)
    }
    set_subset_subring(a, s) = a.subset(s.as_set)
}

/// Set containment in a subring gives containment in its underlying set.
theorem set_as_set_subset_of_subset_subring[R: Ring](a: Set[R], s: Subring[R]) {
    set_subset_subring(a, s) implies a.subset(s.as_set)
} by {
    if set_subset_subring(a, s) {
        set_subset_subring_as_set_eq(a, s)
        a.subset(s.as_set)
    }
}

/// Containment in the underlying set gives set containment in the subring.
theorem set_subset_subring_of_as_set_subset[R: Ring](a: Set[R], s: Subring[R]) {
    a.subset(s.as_set) implies set_subset_subring(a, s)
} by {
    if a.subset(s.as_set) {
        set_subset_subring_as_set_eq(a, s)
        set_subset_subring(a, s)
    }
}

/// The underlying set of a subring is contained in the subring.
theorem subring_as_set_subset_self[R: Ring](s: Subring[R]) {
    set_subset_subring(s.as_set, s)
} by {
    forall(x: R) {
        if s.as_set.contains(x) {
            s.contains(x)
        }
    }
}

/// Set containment is preserved by enlarging the target subring.
theorem set_subset_subring_trans[R: Ring](
    a: Set[R],
    s: Subring[R],
    t: Subring[R]
) {
    set_subset_subring(a, s) and subring_subset(s, t) implies set_subset_subring(a, t)
} by {
    if set_subset_subring(a, s) and subring_subset(s, t) {
        forall(x: R) {
            if a.contains(x) {
                s.contains(x)
                t.contains(x)
            }
        }
    }
}

/// Set containment in a subring is preserved by enlarging the set.
theorem set_subset_subring_of_set_subset[R: Ring](
    a: Set[R],
    b: Set[R],
    s: Subring[R]
) {
    a.subset(b) and set_subset_subring(b, s) implies set_subset_subring(a, s)
} by {
    if a.subset(b) and set_subset_subring(b, s) {
        forall(x: R) {
            if a.contains(x) {
                a.subset(b) = forall(y: R) {
                    a.contains(y) implies b.contains(y)
                }
                b.contains(x)
                s.contains(x)
            }
        }
    }
}

/// True if an element belongs to every subring that contains a given set.
define subring_closure_contains[R: Ring](a: Set[R], x: R) -> Bool {
    forall(s: Subring[R]) {
        set_subset_subring(a, s) implies s.contains(x)
    }
}

/// Membership in the closure gives membership in each subring that contains the set.
theorem subring_closure_contains_of_set_subset_raw[R: Ring](
    a: Set[R],
    s: Subring[R],
    x: R
) {
    subring_closure_contains(a, x) and set_subset_subring(a, s) implies s.contains(x)
} by {
    if subring_closure_contains(a, x) and set_subset_subring(a, s) {
        subring_closure_contains(a, x) = forall(t: Subring[R]) {
            set_subset_subring(a, t) implies t.contains(x)
        }
        s.contains(x)
    }
}

/// The subring closure of a set contains zero.
theorem subring_closure_zero_constraint[R: Ring](a: Set[R]) {
    subring_zero_constraint(subring_closure_contains(a))
} by {
    forall(s: Subring[R]) {
        if set_subset_subring(a, s) {
            subring_contains_zero(s)
            s.contains(R.0)
        }
    }
    subring_closure_contains(a, R.0)
    subring_zero_constraint(subring_closure_contains(a))
}

/// The subring closure of a set contains one.
theorem subring_closure_one_constraint[R: Ring](a: Set[R]) {
    subring_one_constraint(subring_closure_contains(a))
} by {
    forall(s: Subring[R]) {
        if set_subset_subring(a, s) {
            subring_contains_one(s)
            s.contains(R.1)
        }
    }
    subring_closure_contains(a, R.1)
    subring_one_constraint(subring_closure_contains(a))
}

/// The subring closure of a set is closed under addition.
theorem subring_closure_add_constraint[R: Ring](a: Set[R]) {
    subring_add_constraint(subring_closure_contains(a))
} by {
    forall(x: R, y: R) {
        if subring_closure_contains(a, x) and subring_closure_contains(a, y) {
            forall(s: Subring[R]) {
                if set_subset_subring(a, s) {
                    subring_closure_contains_of_set_subset_raw(a, s, x)
                    s.contains(x)
                    subring_closure_contains_of_set_subset_raw(a, s, y)
                    s.contains(y)
                    subring_contains_add(s, x, y)
                    s.contains(x + y)
                }
            }
            subring_closure_contains(a, x + y)
        }
    }
}

/// The subring closure of a set is closed under negation.
theorem subring_closure_neg_constraint[R: Ring](a: Set[R]) {
    subring_neg_constraint(subring_closure_contains(a))
} by {
    forall(x: R) {
        if subring_closure_contains(a, x) {
            forall(s: Subring[R]) {
                if set_subset_subring(a, s) {
                    subring_closure_contains_of_set_subset_raw(a, s, x)
                    s.contains(x)
                    subring_contains_neg(s, x)
                    s.contains(-x)
                }
            }
            subring_closure_contains(a, -x)
        }
    }
}

/// The subring closure of a set is closed under multiplication.
theorem subring_closure_mul_constraint[R: Ring](a: Set[R]) {
    subring_mul_constraint(subring_closure_contains(a))
} by {
    forall(x: R, y: R) {
        if subring_closure_contains(a, x) and subring_closure_contains(a, y) {
            forall(s: Subring[R]) {
                if set_subset_subring(a, s) {
                    subring_closure_contains_of_set_subset_raw(a, s, x)
                    s.contains(x)
                    subring_closure_contains_of_set_subset_raw(a, s, y)
                    s.contains(y)
                    subring_contains_mul(s, x, y)
                    s.contains(x * y)
                }
            }
            subring_closure_contains(a, x * y)
        }
    }
}

/// The subring generated by a set satisfies the subring laws.
theorem subring_generated_constraint[R: Ring](a: Set[R]) {
    subring_constraint(subring_closure_contains(a))
} by {
    subring_closure_zero_constraint(a)
    subring_closure_one_constraint(a)
    subring_closure_add_constraint(a)
    subring_closure_neg_constraint(a)
    subring_closure_mul_constraint(a)
}

/// The smallest subring containing a set.
let subring_closure[R: Ring](a: Set[R]) -> result: Subring[R] satisfy {
    Subring.new(subring_closure_contains(a)) = Option.some(result)
} by {
    subring_generated_constraint(a)
}

/// Membership in the closure means membership in every subring containing the set.
theorem subring_closure_contains_eq[R: Ring](a: Set[R], x: R) {
    subring_closure(a).contains(x) = subring_closure_contains(a, x)
} by {
    subring_closure(a).contains(x) = subring_closure_contains(a, x)
}

/// Every generator belongs to the subring closure.
theorem subring_subset_closure[R: Ring](a: Set[R]) {
    set_subset_subring(a, subring_closure(a))
} by {
    forall(x: R) {
        if a.contains(x) {
            forall(s: Subring[R]) {
                if set_subset_subring(a, s) {
                    s.contains(x)
                }
            }
            subring_closure_contains(a, x)
            subring_closure_contains_eq(a, x)
            subring_closure(a).contains(x)
        }
    }
}

/// A generator is a member of the subring closure.
theorem subring_closure_contains_of_set_contains[R: Ring](a: Set[R], x: R) {
    a.contains(x) implies subring_closure(a).contains(x)
} by {
    if a.contains(x) {
        subring_subset_closure(a)
        subring_closure(a).contains(x)
    }
}

/// The subring closure is contained in any subring containing the set.
theorem subring_closure_subset_of_set_subset[R: Ring](a: Set[R], s: Subring[R]) {
    set_subset_subring(a, s) implies subring_subset(subring_closure(a), s)
} by {
    if set_subset_subring(a, s) {
        forall(x: R) {
            if subring_closure(a).contains(x) {
                subring_closure_contains_eq(a, x)
                subring_closure_contains(a, x)
                subring_closure_contains_of_set_subset_raw(a, s, x)
            }
        }
    }
}

/// The subring closure is the least subring containing the set.
theorem subring_closure_le_iff_set_subset[R: Ring](a: Set[R], s: Subring[R]) {
    subring_subset(subring_closure(a), s) = set_subset_subring(a, s)
} by {
    if subring_subset(subring_closure(a), s) {
        subring_subset_closure(a)
        set_subset_subring_trans(a, subring_closure(a), s)
        set_subset_subring(a, s)
    }
    if set_subset_subring(a, s) {
        subring_closure_subset_of_set_subset(a, s)
        subring_subset(subring_closure(a), s)
    }
    subring_subset(subring_closure(a), s) = set_subset_subring(a, s)
}

/// The subring closure and underlying set maps form a set-containment adjunction.
theorem subring_closure_as_set_galois_connection[R: Ring](a: Set[R], s: Subring[R]) {
    subring_closure(a).as_set.subset(s.as_set) = a.subset(s.as_set)
} by {
    subring_subset_as_set_eq(subring_closure(a), s)
    subring_closure_le_iff_set_subset(a, s)
    set_subset_subring_as_set_eq(a, s)
    subring_closure(a).as_set.subset(s.as_set) = a.subset(s.as_set)
}

/// The closure of the underlying set of a subring is the subring itself.
theorem subring_closure_as_set[R: Ring](s: Subring[R]) {
    subring_closure(s.as_set) = s
} by {
    subring_as_set_subset_self(s)
    subring_closure_subset_of_set_subset(s.as_set, s)
    subring_subset(subring_closure(s.as_set), s)
    subring_subset_closure(s.as_set)
    forall(x: R) {
        if s.contains(x) {
            s.as_set.contains(x)
            subring_closure(s.as_set).contains(x)
        }
    }
    subring_subset(s, subring_closure(s.as_set))
    subring_subset_antisymm(subring_closure(s.as_set), s)
}

/// Subring closure is monotone with respect to set inclusion.
theorem subring_closure_mono[R: Ring](a: Set[R], b: Set[R]) {
    a.subset(b) implies subring_subset(subring_closure(a), subring_closure(b))
} by {
    if a.subset(b) {
        subring_subset_closure(b)
        set_subset_subring_of_set_subset(a, b, subring_closure(b))
        set_subset_subring(a, subring_closure(b))
        subring_closure_subset_of_set_subset(a, subring_closure(b))
        subring_subset(subring_closure(a), subring_closure(b))
    }
}

/// Equal sets have equal subring closures.
theorem subring_closure_eq_of_set_eq[R: Ring](a: Set[R], b: Set[R]) {
    a = b implies subring_closure(a) = subring_closure(b)
} by {
    if a = b {
        subring_closure(a) = subring_closure(b)
    }
}

/// Applying subring closure twice gives the same subring.
theorem subring_closure_idempotent[R: Ring](a: Set[R]) {
    subring_closure(subring_closure(a).as_set) = subring_closure(a)
} by {
    subring_closure_as_set(subring_closure(a))
}

/// The set closure induced by subring generation.
define subring_set_closure[R: Ring](a: Set[R]) -> Set[R] {
    subring_closure(a).as_set
}

/// The subring set closure is the underlying set of the generated subring.
theorem subring_set_closure_at[R: Ring](a: Set[R]) {
    subring_set_closure(a) = subring_closure(a).as_set
}

/// The subring set closure contains the original set.
theorem subring_set_closure_extensive[R: Ring](a: Set[R]) {
    a.subset(subring_set_closure(a))
} by {
    subring_set_closure_at(a)
    subring_subset_closure(a)
    set_subset_subring_as_set_eq(a, subring_closure(a))
    a.subset(subring_closure(a).as_set)
    a.subset(subring_set_closure(a))
}

/// The subring set closure preserves inclusion.
theorem subring_set_closure_mono[R: Ring](a: Set[R], b: Set[R]) {
    a.subset(b) implies subring_set_closure(a).subset(subring_set_closure(b))
} by {
    if a.subset(b) {
        subring_closure_mono(a, b)
        subring_subset(subring_closure(a), subring_closure(b))
        subring_subset_as_set_eq(subring_closure(a), subring_closure(b))
        subring_closure(a).as_set.subset(subring_closure(b).as_set)
        subring_set_closure_at(a)
        subring_set_closure_at(b)
        subring_set_closure(a).subset(subring_set_closure(b))
    }
}

/// The subring set closure is unchanged after two applications.
theorem subring_set_closure_idempotent[R: Ring](a: Set[R]) {
    subring_set_closure(subring_set_closure(a)) = subring_set_closure(a)
} by {
    subring_set_closure_at(a)
    subring_set_closure_at(subring_set_closure(a))
    subring_closure_idempotent(a)
    subring_set_closure(subring_set_closure(a)) = subring_set_closure(a)
}

/// Subring set closure is monotone as a set map.
theorem subring_set_closure_is_monotone[R: Ring] {
    is_subset_monotone_map(subring_set_closure[R])
} by {
    forall(a: Set[R], b: Set[R]) {
        if a.subset(b) {
            subring_set_closure_mono(a, b)
            subring_set_closure(a).subset(subring_set_closure(b))
        }
    }
}

/// Subring set closure is extensive as a set map.
theorem subring_set_closure_is_extensive[R: Ring] {
    is_set_extensive_map(subring_set_closure[R])
} by {
    forall(a: Set[R]) {
        subring_set_closure_extensive(a)
        a.subset(subring_set_closure(a))
    }
}

/// Subring set closure is idempotent as a set map.
theorem subring_set_closure_is_idempotent[R: Ring] {
    is_set_idempotent_map(subring_set_closure[R])
} by {
    forall(a: Set[R]) {
        subring_set_closure_idempotent(a)
        subring_set_closure(subring_set_closure(a)) = subring_set_closure(a)
    }
}

/// Subring generation induces a closure operator on sets.
theorem subring_set_closure_operator[R: Ring] {
    is_set_closure_operator(subring_set_closure[R])
} by {
    subring_set_closure_is_monotone[R]
    subring_set_closure_is_extensive[R]
    subring_set_closure_is_idempotent[R]
    is_set_closure_operator(subring_set_closure[R])
}

/// The closure of the universal set is the full subring.
theorem subring_closure_universal[R: Ring] {
    subring_closure(Set[R].universal_set) = full_subring[R]
} by {
    subring_subset_full(subring_closure(Set[R].universal_set))
    forall(x: R) {
        if full_subring[R].contains(x) {
            Set[R].universal_set.contains(x)
            subring_closure_contains_of_set_contains(Set[R].universal_set, x)
            subring_closure(Set[R].universal_set).contains(x)
        }
    }
    subring_subset(full_subring[R], subring_closure(Set[R].universal_set))
    subring_subset_antisymm(subring_closure(Set[R].universal_set), full_subring[R])
}

/// The least subring containing two subrings.
define subring_sup[R: Ring](a: Subring[R], b: Subring[R]) -> Subring[R] {
    subring_closure(a.as_set.union(b.as_set))
}

/// The join of two subrings is the closure of the union of their underlying sets.
theorem subring_sup_eq_closure_union[R: Ring](a: Subring[R], b: Subring[R]) {
    subring_sup(a, b) = subring_closure(a.as_set.union(b.as_set))
}

/// The left subring is contained in the join.
theorem subring_subset_sup_left[R: Ring](a: Subring[R], b: Subring[R]) {
    subring_subset(a, subring_sup(a, b))
} by {
    forall(x: R) {
        if a.contains(x) {
            subring_as_set_contains_eq(a, x)
            a.as_set.contains(x)
            union_contains_left(a.as_set, b.as_set, x)
            a.as_set.union(b.as_set).contains(x)
            subring_closure_contains_of_set_contains(a.as_set.union(b.as_set), x)
            subring_closure(a.as_set.union(b.as_set)).contains(x)
            subring_sup(a, b).contains(x)
        }
    }
}

/// The right subring is contained in the join.
theorem subring_subset_sup_right[R: Ring](a: Subring[R], b: Subring[R]) {
    subring_subset(b, subring_sup(a, b))
} by {
    forall(x: R) {
        if b.contains(x) {
            subring_as_set_contains_eq(b, x)
            b.as_set.contains(x)
            union_contains_right(a.as_set, b.as_set, x)
            a.as_set.union(b.as_set).contains(x)
            subring_closure_contains_of_set_contains(a.as_set.union(b.as_set), x)
            subring_closure(a.as_set.union(b.as_set)).contains(x)
            subring_sup(a, b).contains(x)
        }
    }
}

/// The join is contained in every common upper bound.
theorem subring_sup_subset_of_subset_left_right[R: Ring](
    a: Subring[R],
    b: Subring[R],
    c: Subring[R]
) {
    subring_subset(a, c) and subring_subset(b, c) implies subring_subset(subring_sup(a, b), c)
} by {
    if subring_subset(a, c) and subring_subset(b, c) {
        forall(x: R) {
            if a.as_set.union(b.as_set).contains(x) {
                union_contains_eq(a.as_set, b.as_set, x)
                if a.as_set.contains(x) {
                    subring_as_set_contains_eq(a, x)
                    a.contains(x)
                    c.contains(x)
                } else {
                    b.as_set.contains(x)
                    subring_as_set_contains_eq(b, x)
                    b.contains(x)
                    c.contains(x)
                }
            }
        }
        set_subset_subring(a.as_set.union(b.as_set), c) = forall(y: R) {
            a.as_set.union(b.as_set).contains(y) implies c.contains(y)
        }
        set_subset_subring(a.as_set.union(b.as_set), c)
        subring_closure_subset_of_set_subset(a.as_set.union(b.as_set), c)
        subring_subset(subring_closure(a.as_set.union(b.as_set)), c)
        subring_subset(subring_sup(a, b), c)
    }
}

/// Containment of a join is equivalent to containment of both subrings.
theorem subring_sup_subset_iff[R: Ring](
    a: Subring[R],
    b: Subring[R],
    c: Subring[R]
) {
    subring_subset(subring_sup(a, b), c) = (subring_subset(a, c) and subring_subset(b, c))
} by {
    if subring_subset(subring_sup(a, b), c) {
        subring_subset_sup_left(a, b)
        subring_subset_trans(a, subring_sup(a, b), c)
        subring_subset(a, c)
        subring_subset_sup_right(a, b)
        subring_subset_trans(b, subring_sup(a, b), c)
        subring_subset(b, c)
        subring_subset(a, c) and subring_subset(b, c)
    }
    if subring_subset(a, c) and subring_subset(b, c) {
        subring_sup_subset_of_subset_left_right(a, b, c)
        subring_subset(subring_sup(a, b), c)
    }
    subring_subset(subring_sup(a, b), c) = (subring_subset(a, c) and subring_subset(b, c))
}

attributes Subring[R: Ring] {
    /// The smallest subring containing a set.
    let closure: Set[R] -> Subring[R] = subring_closure

    /// The least subring containing this subring and another subring.
    let sup: (Subring[R], Subring[R]) -> Subring[R] = subring_sup
}

/// True if a subring is generated by a finite set.
define subring_is_finitely_generated[R: Ring](s: Subring[R]) -> Bool {
    exists(a: Set[R]) {
        a.is_finite and subring_closure(a) = s
    }
}

/// A finitely generated subring has a finite generating set.
theorem subring_is_finitely_generated_witness[R: Ring](s: Subring[R]) {
    subring_is_finitely_generated(s) implies exists(a: Set[R]) {
        a.is_finite and subring_closure(a) = s
    }
}

/// A subring equal to the closure of a finite set is finitely generated.
theorem subring_is_finitely_generated_of_closure_eq[R: Ring](a: Set[R], s: Subring[R]) {
    a.is_finite and subring_closure(a) = s implies subring_is_finitely_generated(s)
} by {
    if a.is_finite and subring_closure(a) = s {
        exists(b: Set[R]) {
            b.is_finite and subring_closure(b) = s
        }
        subring_is_finitely_generated(s)
    }
}

/// Equality preserves finite generation of subrings.
theorem subring_is_finitely_generated_of_eq[R: Ring](s: Subring[R], t: Subring[R]) {
    subring_is_finitely_generated(s) and s = t implies subring_is_finitely_generated(t)
} by {
    if subring_is_finitely_generated(s) and s = t {
        subring_is_finitely_generated_witness(s)
        let a: Set[R] satisfy {
            a.is_finite and subring_closure(a) = s
        }
        subring_closure(a) = t
        subring_is_finitely_generated_of_closure_eq(a, t)
        subring_is_finitely_generated(t)
    }
}

/// The closure of a finite set is a finitely generated subring.
theorem subring_closure_is_finitely_generated[R: Ring](a: Set[R]) {
    a.is_finite implies subring_is_finitely_generated(subring_closure(a))
} by {
    if a.is_finite {
        subring_is_finitely_generated_of_closure_eq(a, subring_closure(a))
    }
}

/// The closure of one element is a finitely generated subring.
theorem subring_closure_singleton_is_finitely_generated[R: Ring](a: R) {
    subring_is_finitely_generated(subring_closure(Set[R].singleton(a)))
} by {
    singleton_set_is_finite(a)
    subring_closure_is_finitely_generated(Set[R].singleton(a))
}

/// The generator belongs to the subring generated by it.
theorem subring_closure_singleton_contains[R: Ring](a: R) {
    subring_closure(Set[R].singleton(a)).contains(a)
} by {
    singleton_contains_eq(a, a)
    subring_closure_contains_of_set_contains(Set[R].singleton(a), a)
}

/// Intersecting the full subring on the left gives the other subring.
theorem full_subring_intersection_left[R: Ring](s: Subring[R]) {
    full_subring[R].intersection(s) = s
} by {
    subring_subset_full(s)
    subring_intersection_eq_right_of_subset(full_subring[R], s)
}

/// Intersecting the full subring on the right gives the other subring.
theorem full_subring_intersection_right[R: Ring](s: Subring[R]) {
    s.intersection(full_subring[R]) = s
} by {
    subring_subset_full(s)
    subring_intersection_eq_left_of_subset(s, full_subring[R])
}

/// The additive subgroup predicate associated to a subring.
theorem subring_add_subgroup_constraint[R: Ring](s: Subring[R]) {
    add_subgroup_constraint(s.contains)
} by {
    subring_contains_zero(s)
    add_zero_constraint(s.contains)
    forall(a: R, b: R) {
        if s.contains(a) and s.contains(b) {
            subring_contains_add(s, a, b)
            s.contains(a + b)
        }
    }
    add_closure_constraint(s.contains)
    forall(a: R) {
        if s.contains(a) {
            subring_contains_neg(s, a)
            s.contains(-a)
        }
    }
    neg_constraint(s.contains)
}

/// The additive subgroup associated to a subring.
let subring_to_add_subgroup[R: Ring](s: Subring[R]) -> result: AddSubgroup[R] satisfy {
    AddSubgroup.new(s.contains) = Option.some(result)
} by {
    subring_add_subgroup_constraint(s)
}

/// The additive subgroup associated to a subring has the same membership predicate.
theorem subring_to_add_subgroup_contains[R: Ring](s: Subring[R]) {
    subring_to_add_subgroup(s).contains = s.contains
}

/// Membership in the additive subgroup associated to a subring is membership in the subring.
theorem subring_to_add_subgroup_contains_eq[R: Ring](s: Subring[R], x: R) {
    subring_to_add_subgroup(s).contains(x) = s.contains(x)
} by {
    subring_to_add_subgroup_contains(s)
    subring_to_add_subgroup(s).contains(x) = s.contains(x)
}

/// Membership in a subring is membership in its associated additive subgroup.
theorem subring_contains_to_add_subgroup_eq[R: Ring](s: Subring[R], x: R) {
    s.contains(x) = subring_to_add_subgroup(s).contains(x)
} by {
    subring_to_add_subgroup_contains_eq(s, x)
    s.contains(x) = subring_to_add_subgroup(s).contains(x)
}

/// The associated additive subgroup has the same underlying set as the subring.
theorem subring_to_add_subgroup_as_set[R: Ring](s: Subring[R]) {
    subring_to_add_subgroup(s).as_set = s.as_set
}

/// The multiplicative submonoid predicate associated to a subring.
theorem subring_submonoid_constraint[R: Ring](s: Subring[R]) {
    submonoid_constraint(s.contains)
} by {
    subring_contains_one(s)
    submonoid_identity_constraint(s.contains)
    forall(a: R, b: R) {
        if s.contains(a) and s.contains(b) {
            subring_contains_mul(s, a, b)
            s.contains(a * b)
        }
    }
    submonoid_closure_constraint(s.contains)
}

/// The multiplicative submonoid associated to a subring.
let subring_to_submonoid[R: Ring](s: Subring[R]) -> result: Submonoid[R] satisfy {
    Submonoid.new(s.contains) = Option.some(result)
} by {
    subring_submonoid_constraint(s)
}

/// The multiplicative submonoid associated to a subring has the same membership predicate.
theorem subring_to_submonoid_contains[R: Ring](s: Subring[R]) {
    subring_to_submonoid(s).contains = s.contains
}

/// Membership in the multiplicative submonoid associated to a subring is membership in the subring.
theorem subring_to_submonoid_contains_eq[R: Ring](s: Subring[R], x: R) {
    subring_to_submonoid(s).contains(x) = s.contains(x)
} by {
    subring_to_submonoid_contains(s)
    subring_to_submonoid(s).contains(x) = s.contains(x)
}

/// Membership in a subring is membership in its associated multiplicative submonoid.
theorem subring_contains_to_submonoid_eq[R: Ring](s: Subring[R], x: R) {
    s.contains(x) = subring_to_submonoid(s).contains(x)
} by {
    subring_to_submonoid_contains_eq(s, x)
    s.contains(x) = subring_to_submonoid(s).contains(x)
}

/// The associated multiplicative submonoid has the same underlying set as the subring.
theorem subring_to_submonoid_as_set[R: Ring](s: Subring[R]) {
    subring_to_submonoid(s).as_set = s.as_set
}

/// The image membership predicate of a subring under a ring homomorphism.
define subring_image_contains[R: Ring, S: Ring](
    f: RingHom[R, S],
    s: Subring[R],
    y: S
) -> Bool {
    exists(x: R) {
        s.contains(x) and f.hom(x) = y
    }
}

/// The image of a subring contains zero.
theorem subring_image_zero_constraint[R: Ring, S: Ring](
    f: RingHom[R, S],
    s: Subring[R]
) {
    subring_zero_constraint(subring_image_contains(f, s))
} by {
    subring_contains_zero(s)
    s.contains(R.0)
    ring_hom_zero(f)
    f.hom(R.0) = S.0
    exists(x: R) {
        s.contains(x) and f.hom(x) = S.0
    }
    subring_image_contains(f, s, S.0)
}

/// The image of a subring contains one.
theorem subring_image_one_constraint[R: Ring, S: Ring](
    f: RingHom[R, S],
    s: Subring[R]
) {
    subring_one_constraint(subring_image_contains(f, s))
} by {
    subring_contains_one(s)
    s.contains(R.1)
    ring_hom_one(f)
    f.hom(R.1) = S.1
    exists(x: R) {
        s.contains(x) and f.hom(x) = S.1
    }
    subring_image_contains(f, s, S.1)
}

/// The image of a subring is closed under addition.
theorem subring_image_add_constraint[R: Ring, S: Ring](
    f: RingHom[R, S],
    s: Subring[R]
) {
    subring_add_constraint(subring_image_contains(f, s))
} by {
    forall(a: S, b: S) {
        if subring_image_contains(f, s, a) and subring_image_contains(f, s, b) {
            let x: R satisfy { s.contains(x) and f.hom(x) = a }
            let y: R satisfy { s.contains(y) and f.hom(y) = b }
            subring_contains_add(s, x, y)
            s.contains(x + y)
            ring_hom_add(f, x, y)
            f.hom(x + y) = f.hom(x) + f.hom(y)
            f.hom(x + y) = a + b
            exists(z: R) {
                s.contains(z) and f.hom(z) = a + b
            }
            subring_image_contains(f, s, a + b)
        }
    }
}

/// The image of a subring is closed under negation.
theorem subring_image_neg_constraint[R: Ring, S: Ring](
    f: RingHom[R, S],
    s: Subring[R]
) {
    subring_neg_constraint(subring_image_contains(f, s))
} by {
    forall(a: S) {
        if subring_image_contains(f, s, a) {
            let x: R satisfy { s.contains(x) and f.hom(x) = a }
            subring_contains_neg(s, x)
            s.contains(-x)
            ring_hom_neg(f, x)
            f.hom(-x) = -f.hom(x)
            f.hom(-x) = -a
            exists(z: R) {
                s.contains(z) and f.hom(z) = -a
            }
            subring_image_contains(f, s, -a)
        }
    }
}

/// The image of a subring is closed under multiplication.
theorem subring_image_mul_constraint[R: Ring, S: Ring](
    f: RingHom[R, S],
    s: Subring[R]
) {
    subring_mul_constraint(subring_image_contains(f, s))
} by {
    forall(a: S, b: S) {
        if subring_image_contains(f, s, a) and subring_image_contains(f, s, b) {
            let x: R satisfy { s.contains(x) and f.hom(x) = a }
            let y: R satisfy { s.contains(y) and f.hom(y) = b }
            subring_contains_mul(s, x, y)
            s.contains(x * y)
            ring_hom_mul(f, x, y)
            f.hom(x * y) = f.hom(x) * f.hom(y)
            f.hom(x * y) = a * b
            exists(z: R) {
                s.contains(z) and f.hom(z) = a * b
            }
            subring_image_contains(f, s, a * b)
        }
    }
}

/// The image of a subring is a subring.
theorem subring_image_constraint[R: Ring, S: Ring](
    f: RingHom[R, S],
    s: Subring[R]
) {
    subring_constraint(subring_image_contains(f, s))
} by {
    subring_image_zero_constraint(f, s)
    subring_image_one_constraint(f, s)
    subring_image_add_constraint(f, s)
    subring_image_neg_constraint(f, s)
    subring_image_mul_constraint(f, s)
}

/// The image of a subring under a ring homomorphism.
let subring_image[R: Ring, S: Ring](f: RingHom[R, S], s: Subring[R]) -> result: Subring[S] satisfy {
    Subring.new(subring_image_contains(f, s)) = Option.some(result)
} by {
    subring_image_constraint(f, s)
}

/// Membership in a subring image means having a source member mapping to the element.
theorem subring_image_contains_eq[R: Ring, S: Ring](
    f: RingHom[R, S],
    s: Subring[R],
    y: S
) {
    subring_image(f, s).contains(y) = exists(x: R) {
        s.contains(x) and f.hom(x) = y
    }
} by {
    Option.some(subring_image(f, s)) = Subring.new(subring_image_contains(f, s))
    subring_image(f, s).contains(y) = subring_image_contains(f, s, y)
    subring_image_contains(f, s, y) = exists(x: R) {
        s.contains(x) and f.hom(x) = y
    }
}

/// An element of a source subring maps into its subring image.
theorem subring_image_contains_hom[R: Ring, S: Ring](
    f: RingHom[R, S],
    s: Subring[R],
    x: R
) {
    s.contains(x) implies subring_image(f, s).contains(f.hom(x))
} by {
    if s.contains(x) {
        f.hom(x) = f.hom(x)
        exists(w: R) {
            s.contains(w) and f.hom(w) = f.hom(x)
        }
        subring_image_contains_eq(f, s, f.hom(x))
        subring_image(f, s).contains(f.hom(x))
    }
}

/// A member of a subring image has a source witness in the original subring.
theorem subring_image_contains_witness[R: Ring, S: Ring](
    f: RingHom[R, S],
    s: Subring[R],
    y: S
) {
    subring_image(f, s).contains(y) implies exists(x: R) {
        s.contains(x) and f.hom(x) = y
    }
} by {
    if subring_image(f, s).contains(y) {
        subring_image_contains_eq(f, s, y)
        exists(x: R) {
            s.contains(x) and f.hom(x) = y
        }
    }
}

/// Subring images are monotone with respect to source subring containment.
theorem subring_image_subset_of_subset[R: Ring, S: Ring](
    f: RingHom[R, S],
    s: Subring[R],
    t: Subring[R]
) {
    subring_subset(s, t) implies subring_subset(subring_image(f, s), subring_image(f, t))
} by {
    if subring_subset(s, t) {
        forall(y: S) {
            if subring_image(f, s).contains(y) {
                subring_image_contains_witness(f, s, y)
                let x: R satisfy { s.contains(x) and f.hom(x) = y }
                subring_subset(s, t) = forall(a: R) {
                    s.contains(a) implies t.contains(a)
                }
                t.contains(x)
                subring_image_contains_eq(f, t, y)
                exists(w: R) {
                    t.contains(w) and f.hom(w) = y
                }
                subring_image(f, t).contains(y)
            }
        }
    }
}

/// The image of a subring under a composite homomorphism is the iterated image.
theorem subring_image_compose[R: Ring, S: Ring, T: Ring](
    f: RingHom[S, T],
    g: RingHom[R, S],
    s: Subring[R]
) {
    subring_image(compose_ring_hom(f, g), s) = subring_image(f, subring_image(g, s))
} by {
    compose_ring_hom_hom(f, g)
    forall(z: T) {
        if subring_image(compose_ring_hom(f, g), s).contains(z) {
            subring_image_contains_witness(compose_ring_hom(f, g), s, z)
            let x: R satisfy { s.contains(x) and compose_ring_hom(f, g).hom(x) = z }
            compose_ring_hom(f, g).hom(x) = f.hom(g.hom(x))
            subring_image_contains_hom(g, s, x)
            subring_image(g, s).contains(g.hom(x))
            exists(y: S) {
                subring_image(g, s).contains(y) and f.hom(y) = z
            }
            subring_image_contains_eq(f, subring_image(g, s), z)
            subring_image(f, subring_image(g, s)).contains(z)
        }
        if subring_image(f, subring_image(g, s)).contains(z) {
            subring_image_contains_witness(f, subring_image(g, s), z)
            let y: S satisfy { subring_image(g, s).contains(y) and f.hom(y) = z }
            subring_image_contains_witness(g, s, y)
            let x: R satisfy { s.contains(x) and g.hom(x) = y }
            compose_ring_hom(f, g).hom(x) = f.hom(g.hom(x))
            compose_ring_hom(f, g).hom(x) = z
            exists(w: R) {
                s.contains(w) and compose_ring_hom(f, g).hom(w) = z
            }
            subring_image_contains_eq(compose_ring_hom(f, g), s, z)
            subring_image(compose_ring_hom(f, g), s).contains(z)
        }
        subring_image(compose_ring_hom(f, g), s).contains(z) =
            subring_image(f, subring_image(g, s)).contains(z)
    }
    subring_ext(subring_image(compose_ring_hom(f, g), s), subring_image(f, subring_image(g, s)))
}

/// The inverse image membership predicate of a subring under a ring homomorphism.
define subring_preimage_contains[R: Ring, S: Ring](
    f: RingHom[R, S],
    t: Subring[S],
    a: R
) -> Bool {
    t.contains(f.hom(a))
}

/// The inverse image of a subring contains zero.
theorem subring_preimage_zero_constraint[R: Ring, S: Ring](
    f: RingHom[R, S],
    t: Subring[S]
) {
    subring_zero_constraint(subring_preimage_contains(f, t))
} by {
    ring_hom_zero(f)
    f.hom(R.0) = S.0
    subring_contains_zero(t)
    t.contains(S.0)
    t.contains(f.hom(R.0))
    subring_preimage_contains(f, t, R.0)
}

/// The inverse image of a subring contains one.
theorem subring_preimage_one_constraint[R: Ring, S: Ring](
    f: RingHom[R, S],
    t: Subring[S]
) {
    subring_one_constraint(subring_preimage_contains(f, t))
} by {
    ring_hom_one(f)
    f.hom(R.1) = S.1
    subring_contains_one(t)
    t.contains(S.1)
    t.contains(f.hom(R.1))
    subring_preimage_contains(f, t, R.1)
}

/// The inverse image of a subring is closed under addition.
theorem subring_preimage_add_constraint[R: Ring, S: Ring](
    f: RingHom[R, S],
    t: Subring[S]
) {
    subring_add_constraint(subring_preimage_contains(f, t))
} by {
    forall(a: R, b: R) {
        if subring_preimage_contains(f, t, a) and subring_preimage_contains(f, t, b) {
            t.contains(f.hom(a))
            t.contains(f.hom(b))
            subring_contains_add(t, f.hom(a), f.hom(b))
            t.contains(f.hom(a) + f.hom(b))
            ring_hom_add(f, a, b)
            f.hom(a + b) = f.hom(a) + f.hom(b)
            t.contains(f.hom(a + b))
            subring_preimage_contains(f, t, a + b)
        }
    }
}

/// The inverse image of a subring is closed under negation.
theorem subring_preimage_neg_constraint[R: Ring, S: Ring](
    f: RingHom[R, S],
    t: Subring[S]
) {
    subring_neg_constraint(subring_preimage_contains(f, t))
} by {
    forall(a: R) {
        if subring_preimage_contains(f, t, a) {
            t.contains(f.hom(a))
            subring_contains_neg(t, f.hom(a))
            t.contains(-f.hom(a))
            ring_hom_neg(f, a)
            f.hom(-a) = -f.hom(a)
            t.contains(f.hom(-a))
            subring_preimage_contains(f, t, -a)
        }
    }
}

/// The inverse image of a subring is closed under multiplication.
theorem subring_preimage_mul_constraint[R: Ring, S: Ring](
    f: RingHom[R, S],
    t: Subring[S]
) {
    subring_mul_constraint(subring_preimage_contains(f, t))
} by {
    forall(a: R, b: R) {
        if subring_preimage_contains(f, t, a) and subring_preimage_contains(f, t, b) {
            t.contains(f.hom(a))
            t.contains(f.hom(b))
            subring_contains_mul(t, f.hom(a), f.hom(b))
            t.contains(f.hom(a) * f.hom(b))
            ring_hom_mul(f, a, b)
            f.hom(a * b) = f.hom(a) * f.hom(b)
            t.contains(f.hom(a * b))
            subring_preimage_contains(f, t, a * b)
        }
    }
}

/// The inverse image of a subring is a subring.
theorem subring_preimage_constraint[R: Ring, S: Ring](
    f: RingHom[R, S],
    t: Subring[S]
) {
    subring_constraint(subring_preimage_contains(f, t))
} by {
    subring_preimage_zero_constraint(f, t)
    subring_preimage_one_constraint(f, t)
    subring_preimage_add_constraint(f, t)
    subring_preimage_neg_constraint(f, t)
    subring_preimage_mul_constraint(f, t)
}

/// The inverse image of a subring under a ring homomorphism.
let subring_preimage[R: Ring, S: Ring](f: RingHom[R, S], t: Subring[S]) -> result: Subring[R] satisfy {
    Subring.new(subring_preimage_contains(f, t)) = Option.some(result)
} by {
    subring_preimage_constraint(f, t)
}

/// Membership in an inverse image means the mapped element belongs to the target subring.
theorem subring_preimage_contains_eq[R: Ring, S: Ring](
    f: RingHom[R, S],
    t: Subring[S],
    a: R
) {
    subring_preimage(f, t).contains(a) = t.contains(f.hom(a))
} by {
    subring_preimage(f, t).contains(a) = subring_preimage_contains(f, t, a)
    subring_preimage_contains(f, t, a) = t.contains(f.hom(a))
}

/// Membership in a preimage follows from membership of the mapped element.
theorem subring_preimage_contains_of_contains_map[R: Ring, S: Ring](
    f: RingHom[R, S],
    t: Subring[S],
    a: R
) {
    t.contains(f.hom(a)) implies subring_preimage(f, t).contains(a)
} by {
    if t.contains(f.hom(a)) {
        subring_preimage_contains_eq(f, t, a)
        subring_preimage(f, t).contains(a)
    }
}

/// Membership in the target follows from membership in the preimage.
theorem subring_contains_map_of_preimage_contains[R: Ring, S: Ring](
    f: RingHom[R, S],
    t: Subring[S],
    a: R
) {
    subring_preimage(f, t).contains(a) implies t.contains(f.hom(a))
} by {
    if subring_preimage(f, t).contains(a) {
        subring_preimage_contains_eq(f, t, a)
        t.contains(f.hom(a))
    }
}

/// Inverse images are monotone with respect to subring containment.
theorem subring_preimage_subset_of_subset[R: Ring, S: Ring](
    f: RingHom[R, S],
    t: Subring[S],
    u: Subring[S]
) {
    subring_subset(t, u) implies subring_subset(subring_preimage(f, t), subring_preimage(f, u))
} by {
    if subring_subset(t, u) {
        forall(a: R) {
            if subring_preimage(f, t).contains(a) {
                subring_preimage_contains_eq(f, t, a)
                t.contains(f.hom(a))
                subring_subset(t, u) = forall(x: S) {
                    t.contains(x) implies u.contains(x)
                }
                u.contains(f.hom(a))
                subring_preimage_contains_eq(f, u, a)
                subring_preimage(f, u).contains(a)
            }
        }
    }
}

/// The inverse image of an intersection is the intersection of the inverse images.
theorem subring_preimage_intersection[R: Ring, S: Ring](
    f: RingHom[R, S],
    t: Subring[S],
    u: Subring[S]
) {
    subring_preimage(f, t.intersection(u)) =
        subring_preimage(f, t).intersection(subring_preimage(f, u))
} by {
    forall(a: R) {
        subring_preimage_contains_eq(f, t.intersection(u), a)
        subring_preimage_contains_eq(f, t, a)
        subring_preimage_contains_eq(f, u, a)
        subring_intersection_contains_eq(t, u, f.hom(a))
        subring_intersection_contains_eq(subring_preimage(f, t), subring_preimage(f, u), a)
        subring_preimage(f, t.intersection(u)).contains(a) =
            subring_preimage(f, t).intersection(subring_preimage(f, u)).contains(a)
    }
    subring_ext(subring_preimage(f, t.intersection(u)),
        subring_preimage(f, t).intersection(subring_preimage(f, u)))
}

/// The inverse image of the full subring is the full subring.
theorem subring_preimage_full[R: Ring, S: Ring](f: RingHom[R, S]) {
    subring_preimage(f, full_subring[S]) = full_subring[R]
} by {
    subring_subset_full(subring_preimage(f, full_subring[S]))
    forall(a: R) {
        if full_subring[R].contains(a) {
            full_subring_contains_everything(f.hom(a))
            subring_preimage_contains_eq(f, full_subring[S], a)
            subring_preimage(f, full_subring[S]).contains(a)
        }
    }
    subring_subset(full_subring[R], subring_preimage(f, full_subring[S]))
    subring_subset_antisymm(subring_preimage(f, full_subring[S]), full_subring[R])
}

/// The inverse image of a subring under a composite homomorphism is the iterated inverse image.
theorem subring_preimage_compose[R: Ring, S: Ring, T: Ring](
    f: RingHom[S, T],
    g: RingHom[R, S],
    u: Subring[T]
) {
    subring_preimage(compose_ring_hom(f, g), u) =
        subring_preimage(g, subring_preimage(f, u))
} by {
    compose_ring_hom_hom(f, g)
    forall(x: R) {
        subring_preimage_contains_eq(compose_ring_hom(f, g), u, x)
        subring_preimage_contains_eq(f, u, g.hom(x))
        subring_preimage_contains_eq(g, subring_preimage(f, u), x)
        compose_ring_hom(f, g).hom(x) = f.hom(g.hom(x))
        subring_preimage(compose_ring_hom(f, g), u).contains(x) =
            subring_preimage(g, subring_preimage(f, u)).contains(x)
    }
    subring_ext(subring_preimage(compose_ring_hom(f, g), u),
        subring_preimage(g, subring_preimage(f, u)))
}

/// Subring image and inverse image form a containment Galois connection.
theorem subring_image_subset_iff_subset_preimage[R: Ring, S: Ring](
    f: RingHom[R, S],
    s: Subring[R],
    t: Subring[S]
) {
    subring_subset(subring_image(f, s), t) = subring_subset(s, subring_preimage(f, t))
} by {
    if subring_subset(subring_image(f, s), t) {
        forall(x: R) {
            if s.contains(x) {
                subring_image_contains_hom(f, s, x)
                subring_image(f, s).contains(f.hom(x))
                subring_subset(subring_image(f, s), t) = forall(y: S) {
                    subring_image(f, s).contains(y) implies t.contains(y)
                }
                t.contains(f.hom(x))
                subring_preimage_contains_eq(f, t, x)
                subring_preimage(f, t).contains(x)
            }
        }
        subring_subset(s, subring_preimage(f, t)) = forall(x: R) {
            s.contains(x) implies subring_preimage(f, t).contains(x)
        }
        subring_subset(s, subring_preimage(f, t))
    }
    if subring_subset(s, subring_preimage(f, t)) {
        forall(y: S) {
            if subring_image(f, s).contains(y) {
                subring_image_contains_witness(f, s, y)
                let x: R satisfy { s.contains(x) and f.hom(x) = y }
                subring_subset(s, subring_preimage(f, t)) = forall(a: R) {
                    s.contains(a) implies subring_preimage(f, t).contains(a)
                }
                subring_preimage(f, t).contains(x)
                subring_contains_map_of_preimage_contains(f, t, x)
                t.contains(f.hom(x))
                t.contains(y)
            }
        }
        subring_subset(subring_image(f, s), t) = forall(y: S) {
            subring_image(f, s).contains(y) implies t.contains(y)
        }
        subring_subset(subring_image(f, s), t)
    }
    subring_subset(subring_image(f, s), t) = subring_subset(s, subring_preimage(f, t))
}

/// The image of a subring under the identity ring homomorphism is itself.
theorem subring_image_identity[R: Ring](s: Subring[R]) {
    subring_image(identity_ring_hom[R], s) = s
} by {
    forall(y: R) {
        if subring_image(identity_ring_hom[R], s).contains(y) {
            subring_image_contains_witness(identity_ring_hom[R], s, y)
            let x: R satisfy { s.contains(x) and identity_ring_hom[R].hom(x) = y }
            identity_ring_hom_hom[R]
            identity_ring_hom[R].hom = identity_fn[R]
            identity_fn[R](x) = x
            identity_ring_hom[R].hom(x) = x
            x = y
            s.contains(y)
        }
        if s.contains(y) {
            identity_ring_hom_hom[R]
            identity_ring_hom[R].hom = identity_fn[R]
            identity_fn[R](y) = y
            identity_ring_hom[R].hom(y) = y
            exists(x: R) {
                s.contains(x) and identity_ring_hom[R].hom(x) = y
            }
            subring_image_contains_eq(identity_ring_hom[R], s, y)
            subring_image(identity_ring_hom[R], s).contains(y)
        }
        subring_image(identity_ring_hom[R], s).contains(y) = s.contains(y)
    }
    subring_ext(subring_image(identity_ring_hom[R], s), s)
}

/// The preimage of a subring under the identity ring homomorphism is itself.
theorem subring_preimage_identity[R: Ring](s: Subring[R]) {
    subring_preimage(identity_ring_hom[R], s) = s
} by {
    forall(x: R) {
        subring_preimage_contains_eq(identity_ring_hom[R], s, x)
        identity_ring_hom_hom[R]
        identity_ring_hom[R].hom = identity_fn[R]
        identity_fn[R](x) = x
        identity_ring_hom[R].hom(x) = x
        subring_preimage(identity_ring_hom[R], s).contains(x) = s.contains(x)
    }
    subring_ext(subring_preimage(identity_ring_hom[R], s), s)
}

/// A subring is contained in the preimage of its image.
theorem subring_subset_preimage_image[R: Ring, S: Ring](f: RingHom[R, S], s: Subring[R]) {
    subring_subset(s, subring_preimage(f, subring_image(f, s)))
} by {
    subring_subset_refl(subring_image(f, s))
    subring_image_subset_iff_subset_preimage(f, s, subring_image(f, s))
    subring_subset(subring_image(f, s), subring_image(f, s)) =
        subring_subset(s, subring_preimage(f, subring_image(f, s)))
    subring_subset(s, subring_preimage(f, subring_image(f, s)))
}

/// The image of a preimage subring is contained in the original target subring.
theorem subring_image_preimage_subset[R: Ring, S: Ring](f: RingHom[R, S], t: Subring[S]) {
    subring_subset(subring_image(f, subring_preimage(f, t)), t)
} by {
    subring_subset_refl(subring_preimage(f, t))
    subring_image_subset_iff_subset_preimage(f, subring_preimage(f, t), t)
    subring_subset(subring_image(f, subring_preimage(f, t)), t) =
        subring_subset(subring_preimage(f, t), subring_preimage(f, t))
    subring_subset(subring_image(f, subring_preimage(f, t)), t)
}

/// Image-preimage-image roundtrip for subrings.
theorem subring_image_preimage_image_eq[R: Ring, S: Ring](f: RingHom[R, S], s: Subring[R]) {
    subring_image(f, subring_preimage(f, subring_image(f, s))) = subring_image(f, s)
} by {
    subring_image_preimage_subset(f, subring_image(f, s))
    subring_subset(subring_image(f, subring_preimage(f, subring_image(f, s))), subring_image(f, s))
    subring_subset_preimage_image(f, s)
    subring_image_subset_of_subset(f, s, subring_preimage(f, subring_image(f, s)))
    subring_subset(subring_image(f, s), subring_image(f, subring_preimage(f, subring_image(f, s))))
    subring_subset_antisymm(subring_image(f, subring_preimage(f, subring_image(f, s))), subring_image(f, s))
}

/// Preimage-image-preimage roundtrip for subrings.
theorem subring_preimage_image_preimage_eq[R: Ring, S: Ring](f: RingHom[R, S], t: Subring[S]) {
    subring_preimage(f, subring_image(f, subring_preimage(f, t))) = subring_preimage(f, t)
} by {
    subring_subset_preimage_image(f, subring_preimage(f, t))
    subring_subset(subring_preimage(f, t), subring_preimage(f, subring_image(f, subring_preimage(f, t))))
    subring_image_preimage_subset(f, t)
    subring_preimage_subset_of_subset(f, subring_image(f, subring_preimage(f, t)), t)
    subring_subset(subring_preimage(f, subring_image(f, subring_preimage(f, t))), subring_preimage(f, t))
    subring_subset_antisymm(subring_preimage(f, subring_image(f, subring_preimage(f, t))), subring_preimage(f, t))
}

/// Subring join is commutative.
theorem subring_sup_comm[R: Ring](a: Subring[R], b: Subring[R]) {
    subring_sup(a, b) = subring_sup(b, a)
} by {
    subring_subset_sup_right(b, a)
    subring_subset_sup_left(b, a)
    subring_sup_subset_of_subset_left_right(a, b, subring_sup(b, a))
    subring_subset_sup_right(a, b)
    subring_subset_sup_left(a, b)
    subring_sup_subset_of_subset_left_right(b, a, subring_sup(a, b))
    subring_subset_antisymm(subring_sup(a, b), subring_sup(b, a))
}

/// Joining a subring with itself gives the same subring.
theorem subring_sup_idempotent[R: Ring](a: Subring[R]) {
    subring_sup(a, a) = a
} by {
    subring_subset_refl(a)
    subring_sup_subset_of_subset_left_right(a, a, a)
    subring_subset_sup_left(a, a)
    subring_subset_antisymm(subring_sup(a, a), a)
}

/// Subring join is associative.
theorem subring_sup_assoc[R: Ring](
    a: Subring[R],
    b: Subring[R],
    c: Subring[R]
) {
    subring_sup(subring_sup(a, b), c) = subring_sup(a, subring_sup(b, c))
} by {
    subring_subset_sup_left(a, subring_sup(b, c))
    subring_subset_sup_right(a, subring_sup(b, c))
    subring_subset_sup_left(b, c)
    subring_subset_sup_right(b, c)
    subring_subset_trans(b, subring_sup(b, c), subring_sup(a, subring_sup(b, c)))
    subring_subset_trans(c, subring_sup(b, c), subring_sup(a, subring_sup(b, c)))
    subring_sup_subset_of_subset_left_right(a, b, subring_sup(a, subring_sup(b, c)))
    subring_sup_subset_of_subset_left_right(subring_sup(a, b), c, subring_sup(a, subring_sup(b, c)))
    subring_subset_sup_left(subring_sup(a, b), c)
    subring_subset_sup_right(subring_sup(a, b), c)
    subring_subset_sup_left(a, b)
    subring_subset_sup_right(a, b)
    subring_subset_trans(a, subring_sup(a, b), subring_sup(subring_sup(a, b), c))
    subring_subset_trans(b, subring_sup(a, b), subring_sup(subring_sup(a, b), c))
    subring_sup_subset_of_subset_left_right(b, c, subring_sup(subring_sup(a, b), c))
    subring_sup_subset_of_subset_left_right(a, subring_sup(b, c), subring_sup(subring_sup(a, b), c))
    subring_subset_antisymm(subring_sup(subring_sup(a, b), c), subring_sup(a, subring_sup(b, c)))
}
