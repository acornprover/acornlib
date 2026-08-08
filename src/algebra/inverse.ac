/// The "Inverse" typeclass represents anything that has a multiplicative inverse operator.
typeclass A: Inverse {
    inverse: A -> A
}
