from algebra.add_comm_monoid import AddCommMonoid
from algebra.add_group import AddGroup, AddGroupHom, is_add_group_hom, add_group_hom_add,
    add_group_hom_zero, add_group_hom_neg, identity_fn_is_add_group_hom,
    compose_is_add_group_hom, inverse_add, inverse_left
from data.basic.functions import identity_fn, compose, compose_assoc, compose_identity_left, compose_identity_right,
    function_eq_transport_predicate, function_eq_transport_predicate_rev

/// AddCommGroup represents an Abelian group. It's a commutative, additive group.
typeclass M: AddCommGroup extends AddCommMonoid, AddGroup {
    // All the properties are provided by the base typeclasses.
}

/// A sum of two differences is the difference of the corresponding sums.
theorem sub_add_sub[A: AddCommGroup](a: A, b: A, c: A, d: A) {
    (a - b) + (c - d) = (a + c) - (b + d)
} by {
    a - b = a + -b
    c - d = c + -d
    (a - b) + (c - d) = a + -b + (c + -d)
    a + -b + (c + -d) = a + c + (-b + -d)
    inverse_add[A](b, d)
    -(b + d) = -d + -b
    -b + -d = -(b + d)
    a + c + (-b + -d) = a + c + -(b + d)
    (a + c) - (b + d) = a + c + -(b + d)
}

/// Adding the subtracted element to a difference recovers the first element.
theorem sub_add_cancel[A: AddCommGroup](a: A, b: A) {
    (a - b) + b = a
} by {
    a - b = a + -b
    (a - b) + b = a + (-b + b)
    inverse_left[A](b)
    -b + b = A.0
    a + A.0 = a
}

/// A zero difference has equal elements.
theorem sub_eq_zero_imp_eq[A: AddCommGroup](a: A, b: A) {
    a - b = A.0 implies a = b
} by {
    if a - b = A.0 {
        sub_add_cancel[A](a, b)
        a = (a - b) + b
        a = A.0 + b
        A.0 + b = b
    }
}

/// True if a function between additive commutative groups preserves addition.
define is_add_comm_group_hom[A: AddCommGroup, B: AddCommGroup](f: A -> B) -> Bool {
    is_add_group_hom(f)
}

/// An additive commutative group homomorphism.
structure AddCommGroupHom[A: AddCommGroup, B: AddCommGroup] {
    /// The mapping of the additive commutative group homomorphism.
    hom: A -> B
} constraint {
    is_add_comm_group_hom(hom)
}

/// Additive commutative group homomorphism extensionality.
theorem add_comm_group_hom_ext[A: AddCommGroup, B: AddCommGroup](
    f: AddCommGroupHom[A, B],
    g: AddCommGroupHom[A, B]
) {
    (forall(a: A) { f.hom(a) = g.hom(a) }) implies f = g
} by {
    if forall(a: A) { f.hom(a) = g.hom(a) } {
        f.hom = g.hom
    }
}

attributes AddCommGroupHom[A: AddCommGroup, B: AddCommGroup] {
    /// Additive commutative group homomorphism extensionality from pointwise equality of the mapping.
    let ext = add_comm_group_hom_ext[A, B]
}

/// Equal additive commutative group homomorphisms have equal underlying functions.
theorem add_comm_group_hom_eq_hom[A: AddCommGroup, B: AddCommGroup](
    f: AddCommGroupHom[A, B],
    g: AddCommGroupHom[A, B]
) {
    f = g implies f.hom = g.hom
}

/// Equal additive commutative group homomorphisms have equal values at every element.
theorem add_comm_group_hom_eq_apply[A: AddCommGroup, B: AddCommGroup](
    f: AddCommGroupHom[A, B],
    g: AddCommGroupHom[A, B],
    a: A
) {
    f = g implies f.hom(a) = g.hom(a)
} by {
    if f = g {
        f.hom(a) = g.hom(a)
    }
}

/// Equality of additive commutative group homomorphisms transports predicates on homomorphisms.
theorem add_comm_group_hom_eq_transport_predicate[A: AddCommGroup, B: AddCommGroup](
    p: AddCommGroupHom[A, B] -> Bool,
    f: AddCommGroupHom[A, B],
    g: AddCommGroupHom[A, B]
) {
    f = g and p(f) implies p(g)
}

/// Equality of additive commutative group homomorphisms transports predicates in the reverse direction.
theorem add_comm_group_hom_eq_transport_predicate_rev[A: AddCommGroup, B: AddCommGroup](
    p: AddCommGroupHom[A, B] -> Bool,
    f: AddCommGroupHom[A, B],
    g: AddCommGroupHom[A, B]
) {
    f = g and p(g) implies p(f)
}

/// Equality of underlying functions determines equality of additive commutative group homomorphisms.
theorem add_comm_group_hom_eq_of_hom_eq[A: AddCommGroup, B: AddCommGroup](
    f: AddCommGroupHom[A, B],
    g: AddCommGroupHom[A, B]
) {
    f.hom = g.hom implies f = g
} by {
    if f.hom = g.hom {
        forall(a: A) {
            f.hom(a) = g.hom(a)
        }
        add_comm_group_hom_ext(f, g)
    }
}

/// Pointwise equality determines equality of additive commutative group homomorphisms.
theorem add_comm_group_hom_eq_of_apply_eq[A: AddCommGroup, B: AddCommGroup](
    f: AddCommGroupHom[A, B],
    g: AddCommGroupHom[A, B]
) {
    (forall(a: A) { f.hom(a) = g.hom(a) }) implies f = g
} by {
    if forall(a: A) { f.hom(a) = g.hom(a) } {
        add_comm_group_hom_ext(f, g)
    }
}

/// An additive commutative group homomorphism is an additive group homomorphism.
theorem add_comm_group_hom_is_add_group_hom[A: AddCommGroup, B: AddCommGroup](f: AddCommGroupHom[A, B]) {
    is_add_group_hom(f.hom)
} by {
}

/// The additive group homomorphism underlying an additive commutative group homomorphism.
let add_comm_group_hom_to_add_group_hom[A: AddCommGroup, B: AddCommGroup](f: AddCommGroupHom[A, B]) -> result: AddGroupHom[A, B] satisfy {
    AddGroupHom.new(f.hom) = Option.some(result)
} by {
    add_comm_group_hom_is_add_group_hom(f)
}

/// The underlying function of the additive group homomorphism associated to an additive commutative group homomorphism.
theorem add_comm_group_hom_to_add_group_hom_hom[A: AddCommGroup, B: AddCommGroup](f: AddCommGroupHom[A, B]) {
    add_comm_group_hom_to_add_group_hom(f).hom = f.hom
}

/// An additive commutative group homomorphism preserves addition.
theorem add_comm_group_hom_add[A: AddCommGroup, B: AddCommGroup](
    f: AddCommGroupHom[A, B],
    a: A,
    b: A
) {
    f.hom(a + b) = f.hom(a) + f.hom(b)
} by {
    let base = add_comm_group_hom_to_add_group_hom(f)
    add_group_hom_add(base, a, b)
}

/// An additive commutative group homomorphism preserves zero.
theorem add_comm_group_hom_zero[A: AddCommGroup, B: AddCommGroup](f: AddCommGroupHom[A, B]) {
    f.hom(A.0) = B.0
} by {
    let base = add_comm_group_hom_to_add_group_hom(f)
    add_group_hom_zero(base)
}

/// An additive commutative group homomorphism preserves additive inverses.
theorem add_comm_group_hom_neg[A: AddCommGroup, B: AddCommGroup](f: AddCommGroupHom[A, B], a: A) {
    f.hom(-a) = -f.hom(a)
} by {
    let base = add_comm_group_hom_to_add_group_hom(f)
    add_group_hom_neg(base, a)
}

/// Equality of functions transports the property of being an additive commutative group homomorphism.
theorem function_eq_transport_add_comm_group_hom[A: AddCommGroup, B: AddCommGroup](f: A -> B, g: A -> B) {
    f = g and is_add_comm_group_hom(f) implies is_add_comm_group_hom(g)
} by {
    if f = g and is_add_comm_group_hom(f) {
        function_eq_transport_predicate(is_add_comm_group_hom[A, B], f, g)
    }
}

/// Equality of functions transports the property of being an additive commutative group homomorphism in reverse.
theorem function_eq_transport_add_comm_group_hom_rev[A: AddCommGroup, B: AddCommGroup](f: A -> B, g: A -> B) {
    f = g and is_add_comm_group_hom(g) implies is_add_comm_group_hom(f)
} by {
    if f = g and is_add_comm_group_hom(g) {
        function_eq_transport_predicate_rev(is_add_comm_group_hom[A, B], f, g)
    }
}

/// The identity function on an additive commutative group preserves additive commutative group structure.
theorem identity_fn_is_add_comm_group_hom[A: AddCommGroup] {
    is_add_comm_group_hom(identity_fn[A])
} by {
    identity_fn_is_add_group_hom[A]
}

/// The composition of additive commutative group homomorphisms preserves structure.
theorem compose_is_add_comm_group_hom[A: AddCommGroup, B: AddCommGroup, C: AddCommGroup](
    f: B -> C,
    g: A -> B
) {
    is_add_comm_group_hom(f) and is_add_comm_group_hom(g) implies
    is_add_comm_group_hom(compose(f, g))
} by {
    if is_add_comm_group_hom(f) and is_add_comm_group_hom(g) {
        is_add_group_hom(g)
        compose_is_add_group_hom(f, g)
        is_add_group_hom(compose(f, g))
    }
}

/// The identity homomorphism on an additive commutative group.
let identity_add_comm_group_hom[A: AddCommGroup]: AddCommGroupHom[A, A] satisfy {
    AddCommGroupHom.new(identity_fn[A]) = Option.some(identity_add_comm_group_hom)
}

/// The underlying function of the identity additive commutative group homomorphism.
theorem identity_add_comm_group_hom_hom[A: AddCommGroup] {
    identity_add_comm_group_hom[A].hom = identity_fn[A]
}

/// The composition of two additive commutative group homomorphisms.
let compose_add_comm_group_hom[A: AddCommGroup, B: AddCommGroup, C: AddCommGroup](f: AddCommGroupHom[B, C], g: AddCommGroupHom[A, B]) -> result: AddCommGroupHom[A, C] satisfy {
    AddCommGroupHom.new(compose(f.hom, g.hom)) = Option.some(result)
} by {
    compose_is_add_comm_group_hom(f.hom, g.hom)
}

/// The underlying function of a composition of additive commutative group homomorphisms.
theorem compose_add_comm_group_hom_hom[A: AddCommGroup, B: AddCommGroup, C: AddCommGroup](
    f: AddCommGroupHom[B, C],
    g: AddCommGroupHom[A, B]
) {
    compose_add_comm_group_hom(f, g).hom = compose(f.hom, g.hom)
}

/// Composition of additive commutative group homomorphisms is associative.
theorem compose_add_comm_group_hom_assoc[A: AddCommGroup, B: AddCommGroup, C: AddCommGroup, D: AddCommGroup](
    f: AddCommGroupHom[C, D],
    g: AddCommGroupHom[B, C],
    h: AddCommGroupHom[A, B]
) {
    compose_add_comm_group_hom(compose_add_comm_group_hom(f, g), h) =
    compose_add_comm_group_hom(f, compose_add_comm_group_hom(g, h))
} by {
    let lhs = compose_add_comm_group_hom(compose_add_comm_group_hom(f, g), h)
    let rhs = compose_add_comm_group_hom(f, compose_add_comm_group_hom(g, h))
    lhs.hom = compose(compose_add_comm_group_hom(f, g).hom, h.hom)
    compose_add_comm_group_hom(f, g).hom = compose(f.hom, g.hom)
    lhs.hom = compose(compose(f.hom, g.hom), h.hom)
    rhs.hom = compose(f.hom, compose_add_comm_group_hom(g, h).hom)
    compose_add_comm_group_hom(g, h).hom = compose(g.hom, h.hom)
    rhs.hom = compose(f.hom, compose(g.hom, h.hom))
    compose_assoc(f.hom, g.hom, h.hom)
    lhs.hom = rhs.hom
}

/// Composing the identity additive commutative group homomorphism on the left leaves a homomorphism unchanged.
theorem compose_add_comm_group_hom_identity_left[A: AddCommGroup, B: AddCommGroup](
    f: AddCommGroupHom[A, B]
) {
    compose_add_comm_group_hom(identity_add_comm_group_hom[B], f) = f
} by {
    let lhs = compose_add_comm_group_hom(identity_add_comm_group_hom[B], f)
    compose_identity_left(f.hom)
}

/// Composing the identity additive commutative group homomorphism on the right leaves a homomorphism unchanged.
theorem compose_add_comm_group_hom_identity_right[A: AddCommGroup, B: AddCommGroup](
    f: AddCommGroupHom[A, B]
) {
    compose_add_comm_group_hom(f, identity_add_comm_group_hom[A]) = f
} by {
    let lhs = compose_add_comm_group_hom(f, identity_add_comm_group_hom[A])
    compose_identity_right(f.hom)
}
