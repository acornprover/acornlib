/// Bundled wrapper for pointwise algebraic operations on functions.

from data.basic.function_algebra import pointwise_add, pointwise_mul, pointwise_zero, pointwise_one,
    pointwise_neg
from data.basic.functions import function_extensionality
from algebra.add import Add
from algebra.zero import Zero
from algebra.neg import Neg
from algebra.mul import Mul
from algebra.one import One
from algebra.add_semigroup import AddSemigroup
from algebra.add_comm_semigroup import AddCommSemigroup
from algebra.add_monoid import AddMonoid
from algebra.add_comm_monoid import AddCommMonoid
from algebra.add_group import AddGroup
from algebra.add_comm_group import AddCommGroup
from algebra.semigroup import Semigroup
from algebra.comm_semigroup import CommSemigroup
from algebra.monoid.monoid import Monoid
from algebra.comm_monoid import CommMonoid
from semiring import Semiring
from algebra.ring.ring import Ring
from comm_ring import CommRing

/// A bundled wrapper around functions `T -> A`, used to install pointwise typeclass instances
/// without registering instances directly on raw function types.
structure PointwiseFunction[T, A] {
    /// The underlying function.
    val: T -> A
}

/// A pointwise function wrapper is determined by its values.
theorem pointwise_function_ext[T, A](f: PointwiseFunction[T, A], g: PointwiseFunction[T, A]) {
    (forall(t: T) { f.val(t) = g.val(t) }) implies f = g
} by {
    if forall(t: T) { f.val(t) = g.val(t) } {
        function_extensionality(f.val, g.val)
        f.val = g.val
    }
}

/// Equal pointwise function wrappers have equal underlying functions.
theorem pointwise_function_eq_val[T, A](f: PointwiseFunction[T, A], g: PointwiseFunction[T, A]) {
    f = g implies f.val = g.val
}

/// Equal pointwise function wrappers have equal values at each input.
theorem pointwise_function_eq_apply[T, A](f: PointwiseFunction[T, A], g: PointwiseFunction[T, A], t: T) {
    f = g implies f.val(t) = g.val(t)
} by {
    if f = g {
        f.val = g.val
        f.val(t) = g.val(t)
    }
}

/// The pointwise sum of two wrapped functions.
define pointwise_function_add[T, A: Add](f: PointwiseFunction[T, A], g: PointwiseFunction[T, A]) -> PointwiseFunction[T, A] {
    PointwiseFunction.new(pointwise_add(f.val, g.val))
}

/// The pointwise product of two wrapped functions.
define pointwise_function_mul[T, A: Mul](f: PointwiseFunction[T, A], g: PointwiseFunction[T, A]) -> PointwiseFunction[T, A] {
    PointwiseFunction.new(pointwise_mul(f.val, g.val))
}

/// The constant zero wrapped function.
let pointwise_function_zero[T, A: Zero]: PointwiseFunction[T, A] =
    PointwiseFunction.new(pointwise_zero[T, A])

/// The constant one wrapped function.
let pointwise_function_one[T, A: One]: PointwiseFunction[T, A] =
    PointwiseFunction.new(pointwise_one[T, A])

/// The pointwise negation of a wrapped function.
define pointwise_function_neg[T, A: Neg](f: PointwiseFunction[T, A]) -> PointwiseFunction[T, A] {
    PointwiseFunction.new(pointwise_neg(f.val))
}

attributes PointwiseFunction[T, A: Add] {
    /// Pointwise addition of wrapped functions.
    define add(self, other: PointwiseFunction[T, A]) -> PointwiseFunction[T, A] {
        pointwise_function_add(self, other)
    }
}

attributes PointwiseFunction[T, A: Zero] {
    /// The constant zero wrapped function.
    let zero: PointwiseFunction[T, A] = pointwise_function_zero[T, A]
}

attributes PointwiseFunction[T, A: Neg] {
    /// Pointwise additive inverse of a wrapped function.
    define neg(self) -> PointwiseFunction[T, A] {
        pointwise_function_neg(self)
    }
}

attributes PointwiseFunction[T, A: Mul] {
    /// Pointwise multiplication of wrapped functions.
    define mul(self, other: PointwiseFunction[T, A]) -> PointwiseFunction[T, A] {
        pointwise_function_mul(self, other)
    }
}

attributes PointwiseFunction[T, A: One] {
    /// The constant one wrapped function.
    let one: PointwiseFunction[T, A] = pointwise_function_one[T, A]
}

/// Wrapped functions have pointwise addition when the codomain does.
instance PointwiseFunction[T, A: Add]: Add {
    let add = PointwiseFunction[T, A].add
}

/// Wrapped functions have a pointwise zero when the codomain does.
instance PointwiseFunction[T, A: Zero]: Zero {
    let 0: PointwiseFunction[T, A] = pointwise_function_zero[T, A]
}

/// Wrapped functions have pointwise negation when the codomain does.
instance PointwiseFunction[T, A: Neg]: Neg {
    let neg = PointwiseFunction[T, A].neg
}

/// Wrapped functions have pointwise multiplication when the codomain does.
instance PointwiseFunction[T, A: Mul]: Mul {
    let mul = PointwiseFunction[T, A].mul
}

/// Wrapped functions have a pointwise one when the codomain does.
instance PointwiseFunction[T, A: One]: One {
    let 1: PointwiseFunction[T, A] = pointwise_function_one[T, A]
}

/// Pointwise addition agrees with addition of values.
theorem pointwise_function_add_apply[T, A: Add](f: PointwiseFunction[T, A], g: PointwiseFunction[T, A], t: T) {
    (f + g).val(t) = f.val(t) + g.val(t)
} by {
    (f + g).val(t) = pointwise_add(f.val, g.val, t)
    pointwise_add(f.val, g.val, t) = f.val(t) + g.val(t)
}

/// The pointwise zero has value zero everywhere.
theorem pointwise_function_zero_apply[T, A: Zero](t: T) {
    Zero.0[PointwiseFunction[T, A]].val(t) = A.0
} by {
    Zero.0[PointwiseFunction[T, A]].val(t) = pointwise_zero[T, A](t)
    pointwise_zero[T, A](t) = A.0
}

/// Pointwise negation agrees with negation of values.
theorem pointwise_function_neg_apply[T, A: Neg](f: PointwiseFunction[T, A], t: T) {
    (-f).val(t) = -f.val(t)
} by {
    (-f).val(t) = pointwise_neg(f.val, t)
    pointwise_neg(f.val, t) = -f.val(t)
}

/// Pointwise multiplication agrees with multiplication of values.
theorem pointwise_function_mul_apply[T, A: Mul](f: PointwiseFunction[T, A], g: PointwiseFunction[T, A], t: T) {
    (f * g).val(t) = f.val(t) * g.val(t)
} by {
    (f * g).val(t) = pointwise_mul(f.val, g.val, t)
    pointwise_mul(f.val, g.val, t) = f.val(t) * g.val(t)
}

/// The pointwise one has value one everywhere.
theorem pointwise_function_one_apply[T, A: One](t: T) {
    One.1[PointwiseFunction[T, A]].val(t) = A.1
} by {
    One.1[PointwiseFunction[T, A]].val(t) = pointwise_one[T, A](t)
    pointwise_one[T, A](t) = A.1
}

/// Pointwise addition on wrapped functions is associative.
theorem pointwise_function_add_associative[T, A: AddSemigroup](
    f: PointwiseFunction[T, A],
    g: PointwiseFunction[T, A],
    h: PointwiseFunction[T, A]
) {
    f + (g + h) = (f + g) + h
} by {
    let lhs = f + (g + h)
    let rhs = (f + g) + h
    forall(t: T) {
        pointwise_function_add_apply(f, g + h, t)
        pointwise_function_add_apply(g, h, t)
        lhs.val(t) = f.val(t) + (g.val(t) + h.val(t))
        pointwise_function_add_apply(f + g, h, t)
        pointwise_function_add_apply(f, g, t)
        rhs.val(t) = (f.val(t) + g.val(t)) + h.val(t)
        f.val(t) + (g.val(t) + h.val(t)) = (f.val(t) + g.val(t)) + h.val(t)
        lhs.val(t) = rhs.val(t)
    }
    pointwise_function_ext(lhs, rhs)
}

/// Pointwise addition on wrapped functions is commutative.
theorem pointwise_function_add_commutative[T, A: AddCommSemigroup](
    f: PointwiseFunction[T, A],
    g: PointwiseFunction[T, A]
) {
    f + g = g + f
} by {
    let lhs = f + g
    let rhs = g + f
    forall(t: T) {
        pointwise_function_add_apply(f, g, t)
        pointwise_function_add_apply(g, f, t)
        f.val(t) + g.val(t) = g.val(t) + f.val(t)
        lhs.val(t) = rhs.val(t)
    }
    pointwise_function_ext(lhs, rhs)
}

/// The pointwise zero is a right identity for pointwise addition.
theorem pointwise_function_add_zero_right[T, A: AddMonoid](f: PointwiseFunction[T, A]) {
    f + Zero.0[PointwiseFunction[T, A]] = f
} by {
    let z = Zero.0[PointwiseFunction[T, A]]
    let lhs = f + z
    forall(t: T) {
        pointwise_function_add_apply(f, z, t)
        pointwise_function_zero_apply[T, A](t)
        lhs.val(t) = f.val(t) + A.0
        f.val(t) + A.0 = f.val(t)
        lhs.val(t) = f.val(t)
    }
    pointwise_function_ext(lhs, f)
}

/// The pointwise zero is a left identity for pointwise addition.
theorem pointwise_function_add_zero_left[T, A: AddMonoid](f: PointwiseFunction[T, A]) {
    Zero.0[PointwiseFunction[T, A]] + f = f
} by {
    let z = Zero.0[PointwiseFunction[T, A]]
    let lhs = z + f
    forall(t: T) {
        pointwise_function_add_apply(z, f, t)
        pointwise_function_zero_apply[T, A](t)
        lhs.val(t) = A.0 + f.val(t)
        A.0 + f.val(t) = f.val(t)
        lhs.val(t) = f.val(t)
    }
    pointwise_function_ext(lhs, f)
}

/// Pointwise negation is a right additive inverse.
theorem pointwise_function_add_neg_right[T, A: AddGroup](f: PointwiseFunction[T, A]) {
    f + -f = Zero.0[PointwiseFunction[T, A]]
} by {
    let lhs = f + -f
    let rhs = Zero.0[PointwiseFunction[T, A]]
    forall(t: T) {
        pointwise_function_add_apply(f, -f, t)
        pointwise_function_neg_apply(f, t)
        pointwise_function_zero_apply[T, A](t)
        lhs.val(t) = f.val(t) + -f.val(t)
        f.val(t) + -f.val(t) = A.0
        lhs.val(t) = rhs.val(t)
    }
    pointwise_function_ext(lhs, rhs)
}

/// Pointwise negation is a left additive inverse.
theorem pointwise_function_add_neg_left[T, A: AddGroup](f: PointwiseFunction[T, A]) {
    -f + f = Zero.0[PointwiseFunction[T, A]]
} by {
    let lhs = -f + f
    let rhs = Zero.0[PointwiseFunction[T, A]]
    forall(t: T) {
        pointwise_function_add_apply(-f, f, t)
        pointwise_function_neg_apply(f, t)
        pointwise_function_zero_apply[T, A](t)
        lhs.val(t) = -f.val(t) + f.val(t)
        -f.val(t) + f.val(t) = A.0
        lhs.val(t) = rhs.val(t)
    }
    pointwise_function_ext(lhs, rhs)
}

/// Pointwise multiplication on wrapped functions is associative.
theorem pointwise_function_mul_associative[T, A: Semigroup](
    f: PointwiseFunction[T, A],
    g: PointwiseFunction[T, A],
    h: PointwiseFunction[T, A]
) {
    f * (g * h) = (f * g) * h
} by {
    let lhs = f * (g * h)
    let rhs = (f * g) * h
    forall(t: T) {
        pointwise_function_mul_apply(f, g * h, t)
        pointwise_function_mul_apply(g, h, t)
        lhs.val(t) = f.val(t) * (g.val(t) * h.val(t))
        pointwise_function_mul_apply(f * g, h, t)
        pointwise_function_mul_apply(f, g, t)
        rhs.val(t) = (f.val(t) * g.val(t)) * h.val(t)
        f.val(t) * (g.val(t) * h.val(t)) = (f.val(t) * g.val(t)) * h.val(t)
        lhs.val(t) = rhs.val(t)
    }
    pointwise_function_ext(lhs, rhs)
}

/// Pointwise multiplication on wrapped functions is commutative.
theorem pointwise_function_mul_commutative[T, A: CommSemigroup](
    f: PointwiseFunction[T, A],
    g: PointwiseFunction[T, A]
) {
    f * g = g * f
} by {
    let lhs = f * g
    let rhs = g * f
    forall(t: T) {
        pointwise_function_mul_apply(f, g, t)
        pointwise_function_mul_apply(g, f, t)
        f.val(t) * g.val(t) = g.val(t) * f.val(t)
        lhs.val(t) = rhs.val(t)
    }
    pointwise_function_ext(lhs, rhs)
}

/// The pointwise one is a right identity for pointwise multiplication.
theorem pointwise_function_mul_one_right[T, A: Monoid](f: PointwiseFunction[T, A]) {
    f * One.1[PointwiseFunction[T, A]] = f
} by {
    let one = One.1[PointwiseFunction[T, A]]
    let lhs = f * one
    forall(t: T) {
        pointwise_function_mul_apply(f, one, t)
        pointwise_function_one_apply[T, A](t)
        lhs.val(t) = f.val(t) * A.1
        f.val(t) * A.1 = f.val(t)
        lhs.val(t) = f.val(t)
    }
    pointwise_function_ext(lhs, f)
}

/// The pointwise one is a left identity for pointwise multiplication.
theorem pointwise_function_mul_one_left[T, A: Monoid](f: PointwiseFunction[T, A]) {
    One.1[PointwiseFunction[T, A]] * f = f
} by {
    let one = One.1[PointwiseFunction[T, A]]
    let lhs = one * f
    forall(t: T) {
        pointwise_function_mul_apply(one, f, t)
        pointwise_function_one_apply[T, A](t)
        lhs.val(t) = A.1 * f.val(t)
        A.1 * f.val(t) = f.val(t)
        lhs.val(t) = f.val(t)
    }
    pointwise_function_ext(lhs, f)
}

/// Pointwise multiplication distributes over pointwise addition from the left.
theorem pointwise_function_mul_add_left[T, A: Semiring](
    f: PointwiseFunction[T, A],
    g: PointwiseFunction[T, A],
    h: PointwiseFunction[T, A]
) {
    f * (g + h) = (f * g) + (f * h)
} by {
    let lhs = f * (g + h)
    let rhs = (f * g) + (f * h)
    forall(t: T) {
        pointwise_function_mul_apply(f, g + h, t)
        pointwise_function_add_apply(g, h, t)
        lhs.val(t) = f.val(t) * (g.val(t) + h.val(t))
        pointwise_function_add_apply(f * g, f * h, t)
        pointwise_function_mul_apply(f, g, t)
        pointwise_function_mul_apply(f, h, t)
        rhs.val(t) = f.val(t) * g.val(t) + f.val(t) * h.val(t)
        f.val(t) * (g.val(t) + h.val(t)) = f.val(t) * g.val(t) + f.val(t) * h.val(t)
        lhs.val(t) = rhs.val(t)
    }
    pointwise_function_ext(lhs, rhs)
}

/// Pointwise multiplication distributes over pointwise addition from the right.
theorem pointwise_function_mul_add_right[T, A: Semiring](
    f: PointwiseFunction[T, A],
    g: PointwiseFunction[T, A],
    h: PointwiseFunction[T, A]
) {
    (f + g) * h = (f * h) + (g * h)
} by {
    let lhs = (f + g) * h
    let rhs = (f * h) + (g * h)
    forall(t: T) {
        pointwise_function_mul_apply(f + g, h, t)
        pointwise_function_add_apply(f, g, t)
        lhs.val(t) = (f.val(t) + g.val(t)) * h.val(t)
        pointwise_function_add_apply(f * h, g * h, t)
        pointwise_function_mul_apply(f, h, t)
        pointwise_function_mul_apply(g, h, t)
        rhs.val(t) = f.val(t) * h.val(t) + g.val(t) * h.val(t)
        (f.val(t) + g.val(t)) * h.val(t) = f.val(t) * h.val(t) + g.val(t) * h.val(t)
        lhs.val(t) = rhs.val(t)
    }
    pointwise_function_ext(lhs, rhs)
}

/// Multiplying on the right by the pointwise zero gives the pointwise zero.
theorem pointwise_function_mul_zero_right[T, A: Semiring](f: PointwiseFunction[T, A]) {
    f * Zero.0[PointwiseFunction[T, A]] = Zero.0[PointwiseFunction[T, A]]
} by {
    let z = Zero.0[PointwiseFunction[T, A]]
    let lhs = f * z
    forall(t: T) {
        pointwise_function_mul_apply(f, z, t)
        pointwise_function_zero_apply[T, A](t)
        lhs.val(t) = f.val(t) * A.0
        f.val(t) * A.0 = A.0
        lhs.val(t) = z.val(t)
    }
    pointwise_function_ext(lhs, z)
}

/// Multiplying on the left by the pointwise zero gives the pointwise zero.
theorem pointwise_function_mul_zero_left[T, A: Semiring](f: PointwiseFunction[T, A]) {
    Zero.0[PointwiseFunction[T, A]] * f = Zero.0[PointwiseFunction[T, A]]
} by {
    let z = Zero.0[PointwiseFunction[T, A]]
    let lhs = z * f
    forall(t: T) {
        pointwise_function_mul_apply(z, f, t)
        pointwise_function_zero_apply[T, A](t)
        lhs.val(t) = A.0 * f.val(t)
        A.0 * f.val(t) = A.0
        lhs.val(t) = z.val(t)
    }
    pointwise_function_ext(lhs, z)
}

/// Typeclass formulation of pointwise addition associativity.
theorem pointwise_function_add_semigroup_law[T, A: AddSemigroup](
    f: PointwiseFunction[T, A],
    g: PointwiseFunction[T, A],
    h: PointwiseFunction[T, A]
) {
    Add.add(f, Add.add(g, h)) = Add.add(Add.add(f, g), h)
} by {
    pointwise_function_add_associative(f, g, h)
}

/// Wrapped functions into an additive semigroup form an additive semigroup.
instance PointwiseFunction[T, A: AddSemigroup]: AddSemigroup

/// Typeclass formulation of pointwise addition commutativity.
theorem pointwise_function_add_comm_semigroup_law[T, A: AddCommSemigroup](
    f: PointwiseFunction[T, A],
    g: PointwiseFunction[T, A]
) {
    Add.add(f, g) = Add.add(g, f)
} by {
    pointwise_function_add_commutative(f, g)
}

/// Wrapped functions into an additive commutative semigroup form an additive commutative semigroup.
instance PointwiseFunction[T, A: AddCommSemigroup]: AddCommSemigroup

/// Typeclass formulation of the right identity law for pointwise addition.
theorem pointwise_function_add_monoid_right_law[T, A: AddMonoid](f: PointwiseFunction[T, A]) {
    Add.add(f, Zero.0[PointwiseFunction[T, A]]) = f
} by {
    pointwise_function_add_zero_right(f)
}

/// Typeclass formulation of the left identity law for pointwise addition.
theorem pointwise_function_add_monoid_left_law[T, A: AddMonoid](f: PointwiseFunction[T, A]) {
    Add.add(Zero.0[PointwiseFunction[T, A]], f) = f
} by {
    pointwise_function_add_zero_left(f)
}

/// Wrapped functions into an additive monoid form an additive monoid.
instance PointwiseFunction[T, A: AddMonoid]: AddMonoid

/// Wrapped functions into an additive commutative monoid form an additive commutative monoid.
instance PointwiseFunction[T, A: AddCommMonoid]: AddCommMonoid

/// Typeclass formulation of the right additive inverse law for pointwise negation.
theorem pointwise_function_add_group_inverse_law[T, A: AddGroup](f: PointwiseFunction[T, A]) {
    Add.add(f, Neg.neg(f)) = Zero.0[PointwiseFunction[T, A]]
} by {
    pointwise_function_add_neg_right(f)
}

/// Wrapped functions into an additive group form an additive group.
instance PointwiseFunction[T, A: AddGroup]: AddGroup

/// Wrapped functions into an additive commutative group form an additive commutative group.
instance PointwiseFunction[T, A: AddCommGroup]: AddCommGroup

/// Typeclass formulation of pointwise multiplication associativity.
theorem pointwise_function_semigroup_law[T, A: Semigroup](
    f: PointwiseFunction[T, A],
    g: PointwiseFunction[T, A],
    h: PointwiseFunction[T, A]
) {
    Mul.mul(f, Mul.mul(g, h)) = Mul.mul(Mul.mul(f, g), h)
} by {
    pointwise_function_mul_associative(f, g, h)
}

/// Wrapped functions into a semigroup form a semigroup under pointwise multiplication.
instance PointwiseFunction[T, A: Semigroup]: Semigroup

/// Typeclass formulation of pointwise multiplication commutativity.
theorem pointwise_function_comm_semigroup_law[T, A: CommSemigroup](
    f: PointwiseFunction[T, A],
    g: PointwiseFunction[T, A]
) {
    Mul.mul(f, g) = Mul.mul(g, f)
} by {
    pointwise_function_mul_commutative(f, g)
}

/// Wrapped functions into a commutative semigroup form a commutative semigroup under pointwise multiplication.
instance PointwiseFunction[T, A: CommSemigroup]: CommSemigroup

/// Typeclass formulation of the right identity law for pointwise multiplication.
theorem pointwise_function_monoid_right_law[T, A: Monoid](f: PointwiseFunction[T, A]) {
    Mul.mul(f, One.1[PointwiseFunction[T, A]]) = f
} by {
    pointwise_function_mul_one_right(f)
}

/// Typeclass formulation of the left identity law for pointwise multiplication.
theorem pointwise_function_monoid_left_law[T, A: Monoid](f: PointwiseFunction[T, A]) {
    Mul.mul(One.1[PointwiseFunction[T, A]], f) = f
} by {
    pointwise_function_mul_one_left(f)
}

/// Wrapped functions into a monoid form a monoid under pointwise multiplication.
instance PointwiseFunction[T, A: Monoid]: Monoid

/// Wrapped functions into a commutative monoid form a commutative monoid under pointwise multiplication.
instance PointwiseFunction[T, A: CommMonoid]: CommMonoid

/// Typeclass formulation of left distributivity for pointwise multiplication.
theorem pointwise_function_semiring_distrib_left_law[T, A: Semiring](
    f: PointwiseFunction[T, A],
    g: PointwiseFunction[T, A],
    h: PointwiseFunction[T, A]
) {
    Mul.mul(f, Add.add(g, h)) = Add.add(Mul.mul(f, g), Mul.mul(f, h))
} by {
    pointwise_function_mul_add_left(f, g, h)
}

/// Typeclass formulation of right distributivity for pointwise multiplication.
theorem pointwise_function_semiring_distrib_right_law[T, A: Semiring](
    f: PointwiseFunction[T, A],
    g: PointwiseFunction[T, A],
    h: PointwiseFunction[T, A]
) {
    Mul.mul(Add.add(f, g), h) = Add.add(Mul.mul(f, h), Mul.mul(g, h))
} by {
    pointwise_function_mul_add_right(f, g, h)
}

/// Typeclass formulation of the right zero law for pointwise multiplication.
theorem pointwise_function_semiring_mul_zero_right_law[T, A: Semiring](f: PointwiseFunction[T, A]) {
    Mul.mul(f, Zero.0[PointwiseFunction[T, A]]) = Zero.0[PointwiseFunction[T, A]]
} by {
    pointwise_function_mul_zero_right(f)
}

/// Typeclass formulation of the left zero law for pointwise multiplication.
theorem pointwise_function_semiring_mul_zero_left_law[T, A: Semiring](f: PointwiseFunction[T, A]) {
    Mul.mul(Zero.0[PointwiseFunction[T, A]], f) = Zero.0[PointwiseFunction[T, A]]
} by {
    pointwise_function_mul_zero_left(f)
}

/// Wrapped functions into a semiring form a semiring under pointwise operations.
instance PointwiseFunction[T, A: Semiring]: Semiring

/// Wrapped functions into a ring form a ring under pointwise operations.
instance PointwiseFunction[T, A: Ring]: Ring

/// Wrapped functions into a commutative ring form a commutative ring under pointwise operations.
instance PointwiseFunction[T, A: CommRing]: CommRing
