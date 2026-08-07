/// Finite generation of submonoids is preserved by binary suprema.

from algebra.monoid.monoid import Monoid
from data.basic.set import Set, union_is_finite_of_finite, sets_subset_union,
    sets_subset_contain_union
from algebra.monoid.submonoid import Submonoid, submonoid_sup, submonoid_closure,
    submonoid_closure_is_finitely_generated,
    set_subset_submonoid, set_as_set_subset_of_subset_submonoid,
    set_subset_submonoid_of_as_set_subset, set_subset_submonoid_trans,
    submonoid_subset_closure, submonoid_closure_mono,
    submonoid_closure_subset_of_set_subset, submonoid_subset,
    submonoid_subset_antisymm,
    submonoid_subset_sup_left, submonoid_subset_sup_right,
    submonoid_sup_subset_of_subset_left_right,
    submonoid_is_finitely_generated, submonoid_is_finitely_generated_witness,
    submonoid_is_finitely_generated_of_closure_eq

/// If two sets lie in a submonoid, then so does their union.
theorem set_subset_submonoid_union[M: Monoid](a: Set[M], b: Set[M], s: Submonoid[M]) {
    set_subset_submonoid(a, s) and set_subset_submonoid(b, s) implies
        set_subset_submonoid(a.union(b), s)
} by {
    if set_subset_submonoid(a, s) and set_subset_submonoid(b, s) {
        set_as_set_subset_of_subset_submonoid(a, s)
        a.subset(s.as_set)
        set_as_set_subset_of_subset_submonoid(b, s)
        b.subset(s.as_set)
        sets_subset_contain_union(a, b, s.as_set)
        a.union(b).subset(s.as_set)
        set_subset_submonoid_of_as_set_subset(a.union(b), s)
        set_subset_submonoid(a.union(b), s)
    }
}

/// The left finite generator set lies in the supremum of the two generated submonoids.
theorem set_subset_submonoid_left_closure_sup[M: Monoid](a: Set[M], b: Set[M]) {
    set_subset_submonoid(a, submonoid_sup(submonoid_closure(a), submonoid_closure(b)))
} by {
    submonoid_subset_closure(a)
    set_subset_submonoid(a, submonoid_closure(a))
    submonoid_subset_sup_left(submonoid_closure(a), submonoid_closure(b))
    submonoid_subset(submonoid_closure(a), submonoid_sup(submonoid_closure(a), submonoid_closure(b)))
    set_subset_submonoid_trans(a, submonoid_closure(a), submonoid_sup(submonoid_closure(a), submonoid_closure(b)))
}

/// The right finite generator set lies in the supremum of the two generated submonoids.
theorem set_subset_submonoid_right_closure_sup[M: Monoid](a: Set[M], b: Set[M]) {
    set_subset_submonoid(b, submonoid_sup(submonoid_closure(a), submonoid_closure(b)))
} by {
    submonoid_subset_closure(b)
    set_subset_submonoid(b, submonoid_closure(b))
    submonoid_subset_sup_right(submonoid_closure(a), submonoid_closure(b))
    submonoid_subset(submonoid_closure(b), submonoid_sup(submonoid_closure(a), submonoid_closure(b)))
    set_subset_submonoid_trans(b, submonoid_closure(b), submonoid_sup(submonoid_closure(a), submonoid_closure(b)))
}

/// The closure of a finite union of generator sets is finitely generated.
theorem submonoid_closure_union_is_finitely_generated[M: Monoid](a: Set[M], b: Set[M]) {
    a.is_finite and b.is_finite implies
        submonoid_is_finitely_generated(submonoid_closure(a.union(b)))
} by {
    if a.is_finite and b.is_finite {
        union_is_finite_of_finite(a, b)
        a.union(b).is_finite
        submonoid_closure_is_finitely_generated(a.union(b))
        submonoid_is_finitely_generated(submonoid_closure(a.union(b)))
    }
}

/// The supremum of closures of finite sets is generated by their finite union.
theorem submonoid_sup_of_closure_finite_sets_eq_closure_union[M: Monoid](a: Set[M], b: Set[M]) {
    submonoid_sup(submonoid_closure(a), submonoid_closure(b)) = submonoid_closure(a.union(b))
} by {
    sets_subset_union(a, b)
    a.subset(a.union(b))
    b.subset(a.union(b))
    submonoid_closure_mono(a, a.union(b))
    submonoid_subset(submonoid_closure(a), submonoid_closure(a.union(b)))
    submonoid_closure_mono(b, a.union(b))
    submonoid_subset(submonoid_closure(b), submonoid_closure(a.union(b)))
    submonoid_sup_subset_of_subset_left_right(
        submonoid_closure(a),
        submonoid_closure(b),
        submonoid_closure(a.union(b))
    )
    submonoid_subset(
        submonoid_sup(submonoid_closure(a), submonoid_closure(b)),
        submonoid_closure(a.union(b))
    )
    set_subset_submonoid_left_closure_sup(a, b)
    set_subset_submonoid(a, submonoid_sup(submonoid_closure(a), submonoid_closure(b)))
    set_subset_submonoid_right_closure_sup(a, b)
    set_subset_submonoid(b, submonoid_sup(submonoid_closure(a), submonoid_closure(b)))
    set_subset_submonoid_union(a, b, submonoid_sup(submonoid_closure(a), submonoid_closure(b)))
    set_subset_submonoid(a.union(b), submonoid_sup(submonoid_closure(a), submonoid_closure(b)))
    submonoid_closure_subset_of_set_subset(
        a.union(b),
        submonoid_sup(submonoid_closure(a), submonoid_closure(b))
    )
    submonoid_subset(
        submonoid_closure(a.union(b)),
        submonoid_sup(submonoid_closure(a), submonoid_closure(b))
    )
    submonoid_subset_antisymm(
        submonoid_sup(submonoid_closure(a), submonoid_closure(b)),
        submonoid_closure(a.union(b))
    )
    submonoid_sup(submonoid_closure(a), submonoid_closure(b)) = submonoid_closure(a.union(b))
}

/// The supremum of submonoids generated by finite sets is finitely generated.
theorem submonoid_sup_of_closure_finite_sets_is_finitely_generated[M: Monoid](a: Set[M], b: Set[M]) {
    a.is_finite and b.is_finite implies
        submonoid_is_finitely_generated(submonoid_sup(submonoid_closure(a), submonoid_closure(b)))
} by {
    if a.is_finite and b.is_finite {
        union_is_finite_of_finite(a, b)
        a.union(b).is_finite
        submonoid_sup_of_closure_finite_sets_eq_closure_union(a, b)
        submonoid_closure(a.union(b)) = submonoid_sup(submonoid_closure(a), submonoid_closure(b))
        submonoid_is_finitely_generated_of_closure_eq(
            a.union(b),
            submonoid_sup(submonoid_closure(a), submonoid_closure(b))
        )
        submonoid_is_finitely_generated(submonoid_sup(submonoid_closure(a), submonoid_closure(b)))
    }
}

/// Finite generation is preserved by binary suprema of submonoids.
theorem submonoid_sup_is_finitely_generated[M: Monoid](s: Submonoid[M], t: Submonoid[M]) {
    submonoid_is_finitely_generated(s) and submonoid_is_finitely_generated(t) implies
        submonoid_is_finitely_generated(submonoid_sup(s, t))
} by {
    if submonoid_is_finitely_generated(s) and submonoid_is_finitely_generated(t) {
        submonoid_is_finitely_generated_witness(s)
        let a: Set[M] satisfy {
            a.is_finite and submonoid_closure(a) = s
        }
        submonoid_is_finitely_generated_witness(t)
        let b: Set[M] satisfy {
            b.is_finite and submonoid_closure(b) = t
        }
        submonoid_sup_of_closure_finite_sets_is_finitely_generated(a, b)
        submonoid_is_finitely_generated(submonoid_sup(submonoid_closure(a), submonoid_closure(b)))
        submonoid_sup(submonoid_closure(a), submonoid_closure(b)) = submonoid_sup(s, t)
        submonoid_is_finitely_generated(submonoid_sup(s, t))
    }
}

/// If two submonoids have chosen generating sets, their supremum is generated by the union.
theorem submonoid_sup_eq_closure_union_of_closure_eq[M: Monoid](
    a: Set[M],
    b: Set[M],
    s: Submonoid[M],
    t: Submonoid[M]
) {
    submonoid_closure(a) = s and submonoid_closure(b) = t implies
        submonoid_sup(s, t) = submonoid_closure(a.union(b))
} by {
    if submonoid_closure(a) = s and submonoid_closure(b) = t {
        submonoid_sup_of_closure_finite_sets_eq_closure_union(a, b)
        submonoid_sup(submonoid_closure(a), submonoid_closure(b)) = submonoid_closure(a.union(b))
        submonoid_sup(submonoid_closure(a), submonoid_closure(b)) = submonoid_sup(s, t)
        submonoid_sup(s, t) = submonoid_closure(a.union(b))
    }
}

/// A supremum with finite chosen generating sets is finitely generated.
theorem submonoid_sup_of_finite_generating_sets_is_finitely_generated[M: Monoid](
    a: Set[M],
    b: Set[M],
    s: Submonoid[M],
    t: Submonoid[M]
) {
    a.is_finite and b.is_finite and submonoid_closure(a) = s and submonoid_closure(b) = t implies
        submonoid_is_finitely_generated(submonoid_sup(s, t))
} by {
    if a.is_finite and b.is_finite and submonoid_closure(a) = s and submonoid_closure(b) = t {
        union_is_finite_of_finite(a, b)
        a.union(b).is_finite
        submonoid_sup_eq_closure_union_of_closure_eq(a, b, s, t)
        submonoid_closure(a.union(b)) = submonoid_sup(s, t)
        submonoid_is_finitely_generated_of_closure_eq(a.union(b), submonoid_sup(s, t))
        submonoid_is_finitely_generated(submonoid_sup(s, t))
    }
}
