from nat import Nat
from algebra.monoid.monoid import Monoid, MonoidHom, monoid_hom_one, monoid_hom_mul
from algebra.subsemigroup import Subsemigroup, subsemigroup_closure_constraint
from data.basic.set import Set, empty_set_is_finite, singleton_contains_eq, singleton_set_is_finite,
    union_contains_eq, union_contains_left, union_contains_right,
    is_subset_monotone_map, is_set_extensive_map, is_set_idempotent_map,
    is_set_closure_operator
from data.basic.functions import predicate_extensionality

// We define submonoids with the "bundled" technique, so a submonoid carries along
// its monoid.

/// True if a subset is closed under the monoid operation.
define submonoid_closure_constraint[M: Monoid](contains: M -> Bool) -> Bool {
    forall(a: M, b: M) {
        contains(a) and contains(b) implies contains(a * b)
    }
}

/// True if a subset contains the identity element of the monoid.
define submonoid_identity_constraint[M: Monoid](contains: M -> Bool) -> Bool {
    contains(M.1)
}

/// True if a subset satisfies all the requirements to be a submonoid.
define submonoid_constraint[M: Monoid](contains: M -> Bool) -> Bool {
    submonoid_identity_constraint(contains) and submonoid_closure_constraint(contains)
}

/// A submonoid of a monoid M, represented as a subset closed under multiplication that contains the identity.
structure Submonoid[M: Monoid] {
    /// True if the given element is a member of this submonoid.
    contains: M -> Bool
} constraint {
    submonoid_constraint(contains)
}

/// Submonoid extensionality: two submonoids are equal when they have the same elements.
theorem submonoid_ext[M: Monoid](a: Submonoid[M], b: Submonoid[M]) {
    (forall(x: M) { a.contains(x) = b.contains(x) }) implies a = b
} by {
    if forall(x: M) { a.contains(x) = b.contains(x) } {
        a.contains = b.contains
    }
}

/// Equal submonoids have equal membership predicates.
theorem submonoid_eq_contains[M: Monoid](a: Submonoid[M], b: Submonoid[M]) {
    a = b implies a.contains = b.contains
}

/// Equal submonoids have equal membership at every element.
theorem submonoid_eq_contains_at[M: Monoid](a: Submonoid[M], b: Submonoid[M], x: M) {
    a = b implies a.contains(x) = b.contains(x)
} by {
    if a = b {
        a.contains = b.contains
        a.contains(x) = b.contains(x)
    }
}

/// Equality of submonoids transports predicates on submonoids.
theorem submonoid_eq_transport_predicate[M: Monoid](
    p: Submonoid[M] -> Bool,
    a: Submonoid[M],
    b: Submonoid[M]
) {
    a = b and p(a) implies p(b)
}

/// Equality of submonoids transports predicates on submonoids in the reverse direction.
theorem submonoid_eq_transport_predicate_rev[M: Monoid](
    p: Submonoid[M] -> Bool,
    a: Submonoid[M],
    b: Submonoid[M]
) {
    a = b and p(b) implies p(a)
}

/// Equality of membership predicates determines equality of submonoids.
theorem submonoid_eq_of_contains_eq[M: Monoid](a: Submonoid[M], b: Submonoid[M]) {
    a.contains = b.contains implies a = b
} by {
    if a.contains = b.contains {
        forall(x: M) {
            a.contains(x) = b.contains(x)
        }
        submonoid_ext(a, b)
    }
}

/// Pointwise equality of membership determines equality of submonoids.
theorem submonoid_eq_of_contains_at_eq[M: Monoid](a: Submonoid[M], b: Submonoid[M]) {
    (forall(x: M) { a.contains(x) = b.contains(x) }) implies a = b
} by {
    if forall(x: M) { a.contains(x) = b.contains(x) } {
        submonoid_ext(a, b)
    }
}

/// The common membership predicate of two submonoids.
define submonoid_intersection_contains[M: Monoid](a: Submonoid[M], b: Submonoid[M], x: M) -> Bool {
    a.contains(x) and b.contains(x)
}

/// The common part of two submonoids contains the identity element.
theorem submonoid_intersection_identity_constraint[M: Monoid](a: Submonoid[M], b: Submonoid[M]) {
    submonoid_identity_constraint(submonoid_intersection_contains(a, b))
} by {
    submonoid_identity_constraint(a.contains)
    submonoid_identity_constraint(b.contains)
    a.contains(M.1)
    b.contains(M.1)
    submonoid_intersection_contains(a, b, M.1)
}

/// The common part of two submonoids is closed under multiplication.
theorem submonoid_intersection_closure_constraint[M: Monoid](a: Submonoid[M], b: Submonoid[M]) {
    submonoid_closure_constraint(submonoid_intersection_contains(a, b))
} by {
    forall(x: M, y: M) {
        if submonoid_intersection_contains(a, b, x) and submonoid_intersection_contains(a, b, y) {
            a.contains(x)
            b.contains(x)
            a.contains(y)
            b.contains(y)
            submonoid_closure_constraint(a.contains)
            submonoid_closure_constraint(b.contains)
            submonoid_closure_constraint(a.contains) = forall(m: M, n: M) {
                a.contains(m) and a.contains(n) implies a.contains(m * n)
            }
            submonoid_closure_constraint(b.contains) = forall(m: M, n: M) {
                b.contains(m) and b.contains(n) implies b.contains(m * n)
            }
            a.contains(x * y)
            b.contains(x * y)
            submonoid_intersection_contains(a, b, x * y)
        }
    }
}

/// The common part of two submonoids is a submonoid.
theorem submonoid_intersection_constraint[M: Monoid](a: Submonoid[M], b: Submonoid[M]) {
    submonoid_constraint(submonoid_intersection_contains(a, b))
} by {
    submonoid_intersection_identity_constraint(a, b)
    submonoid_intersection_closure_constraint(a, b)
}

let submonoid_intersection[M: Monoid](a: Submonoid[M], b: Submonoid[M]) -> result: Submonoid[M] satisfy {
    Submonoid.new(submonoid_intersection_contains(a, b)) = Option.some(result)
} by {
    submonoid_intersection_constraint(a, b)
}

attributes Submonoid[M: Monoid] {
    /// Submonoid extensionality from pointwise equality of membership.
    let ext = submonoid_ext[M]

    /// The subset of monoid elements belonging to this submonoid.
    define as_set(self) -> Set[M] {
        Set[M].new(self.contains)
    }

    /// The common part of two submonoids.
    let intersection: (Submonoid[M], Submonoid[M]) -> Submonoid[M] = submonoid_intersection
}

/// Membership in the underlying set is membership in the submonoid.
theorem submonoid_as_set_contains_eq[M: Monoid](s: Submonoid[M], x: M) {
    s.as_set.contains(x) = s.contains(x)
} by {
    s.as_set.contains(x) = s.contains(x)
}

/// Membership in a submonoid is membership in its underlying set.
theorem submonoid_contains_as_set_eq[M: Monoid](s: Submonoid[M], x: M) {
    s.contains(x) = s.as_set.contains(x)
} by {
    submonoid_as_set_contains_eq(s, x)
    s.contains(x) = s.as_set.contains(x)
}

/// Equal submonoids have equal underlying sets.
theorem submonoid_eq_as_set[M: Monoid](a: Submonoid[M], b: Submonoid[M]) {
    a = b implies a.as_set = b.as_set
}

/// A submonoid is closed under multiplication.
theorem submonoid_mul_mem[M: Monoid](s: Submonoid[M], a: M, b: M) {
    s.contains(a) and s.contains(b) implies s.contains(a * b)
} by {
    if s.contains(a) and s.contains(b) {
        submonoid_closure_constraint(s.contains)
        submonoid_closure_constraint(s.contains) = forall(x: M, y: M) {
            s.contains(x) and s.contains(y) implies s.contains(x * y)
        }
        s.contains(a * b)
    }
}

/// A submonoid is closed under products of three of its elements.
theorem submonoid_mul_mul_mem[M: Monoid](s: Submonoid[M], a: M, b: M, c: M) {
    s.contains(a) and s.contains(b) and s.contains(c) implies s.contains(a * b * c)
} by {
    if s.contains(a) and s.contains(b) and s.contains(c) {
        submonoid_mul_mem(s, a, b)
        s.contains(a * b)
        submonoid_mul_mem(s, a * b, c)
        s.contains(a * b * c)
    }
}

/// A submonoid contains every natural power of each of its elements.
theorem submonoid_pow_mem[M: Monoid](s: Submonoid[M], a: M, n: Nat) {
    s.contains(a) implies s.contains(a.pow(n))
} by {
    define p(k: Nat) -> Bool {
        s.contains(a) implies s.contains(a.pow(k))
    }

    if s.contains(a) {
        a.pow(Nat.0) = M.1
        submonoid_identity_constraint(s.contains)
        s.contains(M.1)
        s.contains(a.pow(Nat.0))
    }
    p(Nat.0)

    forall(k: Nat) {
        if p(k) {
            if s.contains(a) {
                s.contains(a.pow(k))
                submonoid_mul_mem(s, a, a.pow(k))
                s.contains(a * a.pow(k))
                a.pow(k.suc) = a * a.pow(k)
                s.contains(a.pow(k.suc))
            }
            p(k.suc)
        }
    }
    p(n)
    if s.contains(a) {
        s.contains(a.pow(n))
    }
}

/// Membership in the intersection means membership in both submonoids.
theorem submonoid_intersection_contains_eq[M: Monoid](a: Submonoid[M], b: Submonoid[M], x: M) {
    a.intersection(b).contains(x) = (a.contains(x) and b.contains(x))
} by {
    a.intersection(b).contains(x) = submonoid_intersection_contains(a, b, x)
    submonoid_intersection_contains(a, b, x) = (a.contains(x) and b.contains(x))
}

/// The intersection is contained in the left submonoid.
theorem submonoid_intersection_subset_left[M: Monoid](a: Submonoid[M], b: Submonoid[M], x: M) {
    a.intersection(b).contains(x) implies a.contains(x)
} by {
    if a.intersection(b).contains(x) {
        submonoid_intersection_contains_eq(a, b, x)
        a.contains(x)
    }
}

/// The intersection is contained in the right submonoid.
theorem submonoid_intersection_subset_right[M: Monoid](a: Submonoid[M], b: Submonoid[M], x: M) {
    a.intersection(b).contains(x) implies b.contains(x)
} by {
    if a.intersection(b).contains(x) {
        submonoid_intersection_contains_eq(a, b, x)
        b.contains(x)
    }
}

/// Submonoid intersection is commutative.
theorem submonoid_intersection_comm[M: Monoid](a: Submonoid[M], b: Submonoid[M]) {
    a.intersection(b) = b.intersection(a)
} by {
    forall(x: M) {
        if a.intersection(b).contains(x) {
            submonoid_intersection_contains_eq(a, b, x)
            a.contains(x)
            b.contains(x)
            submonoid_intersection_contains_eq(b, a, x)
            b.intersection(a).contains(x)
        }
        if b.intersection(a).contains(x) {
            submonoid_intersection_contains_eq(b, a, x)
            b.contains(x)
            a.contains(x)
            submonoid_intersection_contains_eq(a, b, x)
            a.intersection(b).contains(x)
        }
        a.intersection(b).contains(x) = b.intersection(a).contains(x)
    }
    submonoid_ext(a.intersection(b), b.intersection(a))
}

/// Submonoid intersection is associative.
theorem submonoid_intersection_assoc[M: Monoid](
    a: Submonoid[M],
    b: Submonoid[M],
    c: Submonoid[M]
) {
    a.intersection(b).intersection(c) = a.intersection(b.intersection(c))
} by {
    let lhs = a.intersection(b).intersection(c)
    let rhs = a.intersection(b.intersection(c))
    forall(x: M) {
        if lhs.contains(x) {
            submonoid_intersection_contains_eq(a.intersection(b), c, x)
            a.intersection(b).contains(x)
            c.contains(x)
            submonoid_intersection_contains_eq(a, b, x)
            a.contains(x)
            b.contains(x)
            submonoid_intersection_contains_eq(b, c, x)
            b.intersection(c).contains(x)
            submonoid_intersection_contains_eq(a, b.intersection(c), x)
            rhs.contains(x)
        }
        if rhs.contains(x) {
            submonoid_intersection_contains_eq(a, b.intersection(c), x)
            a.contains(x)
            b.intersection(c).contains(x)
            submonoid_intersection_contains_eq(b, c, x)
            b.contains(x)
            c.contains(x)
            submonoid_intersection_contains_eq(a, b, x)
            a.intersection(b).contains(x)
            submonoid_intersection_contains_eq(a.intersection(b), c, x)
            lhs.contains(x)
        }
        lhs.contains(x) = rhs.contains(x)
    }
    submonoid_ext(lhs, rhs)
}

/// Submonoid intersection is idempotent.
theorem submonoid_intersection_idempotent[M: Monoid](a: Submonoid[M]) {
    a.intersection(a) = a
} by {
    forall(x: M) {
        if a.intersection(a).contains(x) {
            submonoid_intersection_contains_eq(a, a, x)
            a.contains(x)
        }
        if a.contains(x) {
            submonoid_intersection_contains_eq(a, a, x)
            a.intersection(a).contains(x)
        }
        a.intersection(a).contains(x) = a.contains(x)
    }
    submonoid_ext(a.intersection(a), a)
}

/// True if every element of one submonoid belongs to another.
define submonoid_subset[M: Monoid](a: Submonoid[M], b: Submonoid[M]) -> Bool {
    forall(x: M) {
        a.contains(x) implies b.contains(x)
    }
}

/// Submonoid containment is containment of the underlying sets.
theorem submonoid_subset_as_set_eq[M: Monoid](a: Submonoid[M], b: Submonoid[M]) {
    submonoid_subset(a, b) = a.as_set.subset(b.as_set)
} by {
    if submonoid_subset(a, b) {
        forall(x: M) {
            if a.as_set.contains(x) {
                submonoid_as_set_contains_eq(a, x)
                a.contains(x)
                b.contains(x)
                submonoid_as_set_contains_eq(b, x)
                b.as_set.contains(x)
            }
        }
        a.as_set.subset(b.as_set)
    }
    if a.as_set.subset(b.as_set) {
        forall(x: M) {
            if a.contains(x) {
                submonoid_as_set_contains_eq(a, x)
                a.as_set.contains(x)
                a.as_set.subset(b.as_set) = forall(y: M) {
                    a.as_set.contains(y) implies b.as_set.contains(y)
                }
                b.as_set.contains(x)
                submonoid_as_set_contains_eq(b, x)
                b.contains(x)
            }
        }
        submonoid_subset(a, b)
    }
    submonoid_subset(a, b) = a.as_set.subset(b.as_set)
}

/// Submonoid containment gives containment of underlying sets.
theorem submonoid_as_set_subset_of_subset[M: Monoid](a: Submonoid[M], b: Submonoid[M]) {
    submonoid_subset(a, b) implies a.as_set.subset(b.as_set)
} by {
    if submonoid_subset(a, b) {
        submonoid_subset_as_set_eq(a, b)
        a.as_set.subset(b.as_set)
    }
}

/// Containment of underlying sets gives submonoid containment.
theorem submonoid_subset_of_as_set_subset[M: Monoid](a: Submonoid[M], b: Submonoid[M]) {
    a.as_set.subset(b.as_set) implies submonoid_subset(a, b)
} by {
    if a.as_set.subset(b.as_set) {
        submonoid_subset_as_set_eq(a, b)
        submonoid_subset(a, b)
    }
}

/// Submonoid inclusion is reflexive.
theorem submonoid_subset_refl[M: Monoid](a: Submonoid[M]) {
    submonoid_subset(a, a)
} by {
    forall(x: M) {
        a.contains(x) implies a.contains(x)
    }
}

/// Submonoid inclusion is transitive.
theorem submonoid_subset_trans[M: Monoid](
    a: Submonoid[M],
    b: Submonoid[M],
    c: Submonoid[M]
) {
    submonoid_subset(a, b) and submonoid_subset(b, c) implies submonoid_subset(a, c)
} by {
    if submonoid_subset(a, b) and submonoid_subset(b, c) {
        forall(x: M) {
            if a.contains(x) {
                b.contains(x)
                c.contains(x)
            }
        }
    }
}

/// The intersection of two submonoids is contained in the left submonoid.
theorem submonoid_intersection_subset_left_relation[M: Monoid](
    a: Submonoid[M],
    b: Submonoid[M]
) {
    submonoid_subset(a.intersection(b), a)
} by {
    forall(x: M) {
        if a.intersection(b).contains(x) {
            submonoid_intersection_contains_eq(a, b, x)
            a.contains(x)
        }
    }
}

/// The intersection of two submonoids is contained in the right submonoid.
theorem submonoid_intersection_subset_right_relation[M: Monoid](
    a: Submonoid[M],
    b: Submonoid[M]
) {
    submonoid_subset(a.intersection(b), b)
} by {
    forall(x: M) {
        if a.intersection(b).contains(x) {
            submonoid_intersection_contains_eq(a, b, x)
            b.contains(x)
        }
    }
}

/// Every common submonoid of two submonoids is contained in their intersection.
theorem submonoid_subset_intersection_of_subset_left_right[M: Monoid](
    c: Submonoid[M],
    a: Submonoid[M],
    b: Submonoid[M]
) {
    submonoid_subset(c, a) and submonoid_subset(c, b) implies submonoid_subset(c, a.intersection(b))
} by {
    if submonoid_subset(c, a) and submonoid_subset(c, b) {
        forall(x: M) {
            if c.contains(x) {
                a.contains(x)
                b.contains(x)
                submonoid_intersection_contains_eq(a, b, x)
                a.intersection(b).contains(x)
            }
        }
    }
}

/// Containment in an intersection is equivalent to containment in both submonoids.
theorem submonoid_subset_intersection_iff[M: Monoid](
    c: Submonoid[M],
    a: Submonoid[M],
    b: Submonoid[M]
) {
    submonoid_subset(c, a.intersection(b)) = (submonoid_subset(c, a) and submonoid_subset(c, b))
} by {
    if submonoid_subset(c, a.intersection(b)) {
        submonoid_intersection_subset_left_relation(a, b)
        submonoid_subset_trans(c, a.intersection(b), a)
        submonoid_subset(c, a)
        submonoid_intersection_subset_right_relation(a, b)
        submonoid_subset_trans(c, a.intersection(b), b)
        submonoid_subset(c, b)
        submonoid_subset(c, a) and submonoid_subset(c, b)
    }
    if submonoid_subset(c, a) and submonoid_subset(c, b) {
        submonoid_subset_intersection_of_subset_left_right(c, a, b)
        submonoid_subset(c, a.intersection(b))
    }
    submonoid_subset(c, a.intersection(b)) = (submonoid_subset(c, a) and submonoid_subset(c, b))
}

/// True if an element is the identity element of the monoid.
define is_monoid_identity[M: Monoid](m: M) -> Bool {
    m = M.1
}

theorem identity_submonoid_constraint[M: Monoid] {
    submonoid_constraint(is_monoid_identity[M])
} by {
    is_monoid_identity(M.1)
    submonoid_identity_constraint(is_monoid_identity[M])
    forall(a: M, b: M) {
        if is_monoid_identity(a) and is_monoid_identity(b) {
            a = M.1
            b = M.1
            M.1 * M.1 = M.1
            is_monoid_identity(a * b)
        }
    }
    submonoid_closure_constraint(is_monoid_identity[M])
}

/// The trivial submonoid containing only the identity element.
let identity_submonoid[M: Monoid]: Submonoid[M] satisfy {
    Submonoid.new(is_monoid_identity[M]) = Option.some(identity_submonoid)
}

theorem identity_submonoid_only_has_identity[M: Monoid](m: M) {
    identity_submonoid[M].contains(m) implies m = M.1
}

/// Every submonoid contains the identity element.
theorem submonoid_contains_identity[M: Monoid](s: Submonoid[M]) {
    s.contains(M.1)
}

/// Membership in the identity submonoid is the same as equality to the identity element.
theorem identity_submonoid_contains_eq[M: Monoid](m: M) {
    identity_submonoid[M].contains(m) = (m = M.1)
} by {
    if identity_submonoid[M].contains(m) {
        identity_submonoid_only_has_identity(m)
        m = M.1
    }
    if m = M.1 {
        submonoid_contains_identity(identity_submonoid[M])
        identity_submonoid[M].contains(M.1)
        identity_submonoid[M].contains(m)
    }
    identity_submonoid[M].contains(m) = (m = M.1)
}

/// The identity submonoid is contained in every submonoid.
theorem identity_submonoid_subset[M: Monoid](s: Submonoid[M]) {
    submonoid_subset(identity_submonoid[M], s)
} by {
    forall(x: M) {
        if identity_submonoid[M].contains(x) {
            identity_submonoid_only_has_identity(x)
            x = M.1
            submonoid_contains_identity(s)
            s.contains(M.1)
            s.contains(x)
        }
    }
}

/// The full subset of any monoid is closed under multiplication and contains the identity.
define full_monoid_contains[M: Monoid](a: M) -> Bool {
    true
}

theorem full_submonoid_constraint[M: Monoid] {
    submonoid_constraint(full_monoid_contains[M])
} by {
    full_monoid_contains[M](M.1)
    submonoid_identity_constraint(full_monoid_contains[M])
    forall(a: M, b: M) {
        if full_monoid_contains[M](a) and full_monoid_contains[M](b) {
            full_monoid_contains[M](a * b)
        }
    }
    submonoid_closure_constraint(full_monoid_contains[M])
}

/// The full submonoid, containing every element of the monoid.
let full_submonoid[M: Monoid]: Submonoid[M] satisfy {
    Submonoid.new(full_monoid_contains[M]) = Option.some(full_submonoid)
}

theorem full_submonoid_contains_everything[M: Monoid](m: M) {
    full_submonoid[M].contains(m)
}

/// Membership in the full submonoid is always true.
theorem full_submonoid_contains_eq[M: Monoid](m: M) {
    full_submonoid[M].contains(m) = true
} by {
    full_submonoid_contains_everything(m)
}

/// Every submonoid is contained in the full submonoid.
theorem submonoid_subset_full[M: Monoid](s: Submonoid[M]) {
    submonoid_subset(s, full_submonoid[M])
} by {
    forall(x: M) {
        if s.contains(x) {
            full_submonoid_contains_everything(x)
        }
    }
}

/// Mutual inclusion of submonoids forces equality.
theorem submonoid_subset_antisymm[M: Monoid](a: Submonoid[M], b: Submonoid[M]) {
    submonoid_subset(a, b) and submonoid_subset(b, a) implies a = b
} by {
    if submonoid_subset(a, b) and submonoid_subset(b, a) {
        submonoid_subset(a, b) = forall(y: M) {
            a.contains(y) implies b.contains(y)
        }
        submonoid_subset(b, a) = forall(y: M) {
            b.contains(y) implies a.contains(y)
        }
        forall(x: M) {
            if a.contains(x) {
                b.contains(x)
            }
            if b.contains(x) {
                a.contains(x)
            }
            a.contains(x) = b.contains(x)
        }
        predicate_extensionality(a.contains, b.contains)
        a.contains = b.contains
        a = b
    }
}

/// Equality of submonoids is equivalent to mutual inclusion.
theorem submonoid_eq_iff_subset_both[M: Monoid](a: Submonoid[M], b: Submonoid[M]) {
    a = b = (submonoid_subset(a, b) and submonoid_subset(b, a))
} by {
    if a = b {
        submonoid_subset_refl(a)
        submonoid_subset(a, b)
        submonoid_subset(b, a)
        submonoid_subset(a, b) and submonoid_subset(b, a)
    }
    if submonoid_subset(a, b) and submonoid_subset(b, a) {
        submonoid_subset_antisymm(a, b)
        a = b
    }
    a = b = (submonoid_subset(a, b) and submonoid_subset(b, a))
}

/// Intersecting with a larger submonoid gives the smaller submonoid.
theorem submonoid_intersection_eq_left_of_subset[M: Monoid](a: Submonoid[M], b: Submonoid[M]) {
    submonoid_subset(a, b) implies a.intersection(b) = a
} by {
    if submonoid_subset(a, b) {
        submonoid_intersection_subset_left_relation(a, b)
        submonoid_subset_intersection_of_subset_left_right(a, a, b)
        submonoid_subset(a, a.intersection(b))
        submonoid_subset_antisymm(a.intersection(b), a)
        a.intersection(b) = a
    }
}

/// Intersecting with a larger submonoid on the left gives the smaller submonoid.
theorem submonoid_intersection_eq_right_of_subset[M: Monoid](a: Submonoid[M], b: Submonoid[M]) {
    submonoid_subset(b, a) implies a.intersection(b) = b
} by {
    if submonoid_subset(b, a) {
        submonoid_intersection_subset_right_relation(a, b)
        submonoid_subset_intersection_of_subset_left_right(b, a, b)
        submonoid_subset(b, a.intersection(b))
        submonoid_subset_antisymm(a.intersection(b), b)
        a.intersection(b) = b
    }
}

/// Intersecting the identity submonoid on the left gives the identity submonoid.
theorem identity_submonoid_intersection_left[M: Monoid](s: Submonoid[M]) {
    identity_submonoid[M].intersection(s) = identity_submonoid[M]
} by {
    identity_submonoid_subset(s)
    submonoid_intersection_eq_left_of_subset(identity_submonoid[M], s)
}

/// Intersecting the identity submonoid on the right gives the identity submonoid.
theorem identity_submonoid_intersection_right[M: Monoid](s: Submonoid[M]) {
    s.intersection(identity_submonoid[M]) = identity_submonoid[M]
} by {
    identity_submonoid_subset(s)
    submonoid_intersection_eq_right_of_subset(s, identity_submonoid[M])
}

/// Intersecting the full submonoid on the left gives the other submonoid.
theorem full_submonoid_intersection_left[M: Monoid](s: Submonoid[M]) {
    full_submonoid[M].intersection(s) = s
} by {
    submonoid_subset_full(s)
    submonoid_intersection_eq_right_of_subset(full_submonoid[M], s)
}

/// Intersecting the full submonoid on the right gives the other submonoid.
theorem full_submonoid_intersection_right[M: Monoid](s: Submonoid[M]) {
    s.intersection(full_submonoid[M]) = s
} by {
    submonoid_subset_full(s)
    submonoid_intersection_eq_left_of_subset(s, full_submonoid[M])
}

/// True if every element of a set belongs to a submonoid.
define set_subset_submonoid[M: Monoid](a: Set[M], s: Submonoid[M]) -> Bool {
    forall(x: M) {
        a.contains(x) implies s.contains(x)
    }
}

/// Set containment in a submonoid is containment in its underlying set.
theorem set_subset_submonoid_as_set_eq[M: Monoid](a: Set[M], s: Submonoid[M]) {
    set_subset_submonoid(a, s) = a.subset(s.as_set)
} by {
    if set_subset_submonoid(a, s) {
        forall(x: M) {
            if a.contains(x) {
                s.contains(x)
                submonoid_as_set_contains_eq(s, x)
                s.as_set.contains(x)
            }
        }
        a.subset(s.as_set)
    }
    if a.subset(s.as_set) {
        forall(x: M) {
            if a.contains(x) {
                a.subset(s.as_set) = forall(y: M) {
                    a.contains(y) implies s.as_set.contains(y)
                }
                s.as_set.contains(x)
                submonoid_as_set_contains_eq(s, x)
                s.contains(x)
            }
        }
        set_subset_submonoid(a, s)
    }
    set_subset_submonoid(a, s) = a.subset(s.as_set)
}

/// Set containment in a submonoid gives containment in its underlying set.
theorem set_as_set_subset_of_subset_submonoid[M: Monoid](a: Set[M], s: Submonoid[M]) {
    set_subset_submonoid(a, s) implies a.subset(s.as_set)
} by {
    if set_subset_submonoid(a, s) {
        set_subset_submonoid_as_set_eq(a, s)
        a.subset(s.as_set)
    }
}

/// Containment in the underlying set gives set containment in the submonoid.
theorem set_subset_submonoid_of_as_set_subset[M: Monoid](a: Set[M], s: Submonoid[M]) {
    a.subset(s.as_set) implies set_subset_submonoid(a, s)
} by {
    if a.subset(s.as_set) {
        set_subset_submonoid_as_set_eq(a, s)
        set_subset_submonoid(a, s)
    }
}

/// The underlying set of a submonoid is contained in the submonoid.
theorem submonoid_as_set_subset_self[M: Monoid](s: Submonoid[M]) {
    set_subset_submonoid(s.as_set, s)
} by {
    forall(x: M) {
        if s.as_set.contains(x) {
            s.contains(x)
        }
    }
}

/// Set containment is preserved by enlarging the target submonoid.
theorem set_subset_submonoid_trans[M: Monoid](
    a: Set[M],
    s: Submonoid[M],
    t: Submonoid[M]
) {
    set_subset_submonoid(a, s) and submonoid_subset(s, t) implies set_subset_submonoid(a, t)
} by {
    if set_subset_submonoid(a, s) and submonoid_subset(s, t) {
        forall(x: M) {
            if a.contains(x) {
                s.contains(x)
                t.contains(x)
            }
        }
    }
}

/// Set containment in a submonoid is preserved by enlarging the set.
theorem set_subset_submonoid_of_set_subset[M: Monoid](
    a: Set[M],
    b: Set[M],
    s: Submonoid[M]
) {
    a.subset(b) and set_subset_submonoid(b, s) implies set_subset_submonoid(a, s)
} by {
    if a.subset(b) and set_subset_submonoid(b, s) {
        forall(x: M) {
            if a.contains(x) {
                a.subset(b) = forall(y: M) {
                    a.contains(y) implies b.contains(y)
                }
                b.contains(x)
                s.contains(x)
            }
        }
    }
}

/// True if an element belongs to every submonoid that contains a given set.
define submonoid_closure_contains[M: Monoid](a: Set[M], x: M) -> Bool {
    forall(s: Submonoid[M]) {
        set_subset_submonoid(a, s) implies s.contains(x)
    }
}

/// Membership in the closure gives membership in each submonoid that contains the set.
theorem submonoid_closure_contains_of_set_subset_raw[M: Monoid](
    a: Set[M],
    s: Submonoid[M],
    x: M
) {
    submonoid_closure_contains(a, x) and set_subset_submonoid(a, s) implies s.contains(x)
} by {
    if submonoid_closure_contains(a, x) and set_subset_submonoid(a, s) {
        submonoid_closure_contains(a, x) = forall(t: Submonoid[M]) {
            set_subset_submonoid(a, t) implies t.contains(x)
        }
        s.contains(x)
    }
}

/// The submonoid closure of a set contains the identity.
theorem submonoid_closure_identity_constraint[M: Monoid](a: Set[M]) {
    submonoid_identity_constraint(submonoid_closure_contains(a))
} by {
    forall(s: Submonoid[M]) {
        if set_subset_submonoid(a, s) {
            submonoid_contains_identity(s)
            s.contains(M.1)
        }
    }
    submonoid_closure_contains(a, M.1)
    submonoid_identity_constraint(submonoid_closure_contains(a))
}

/// The submonoid closure of a set is closed under multiplication.
theorem submonoid_closure_closure_constraint[M: Monoid](a: Set[M]) {
    submonoid_closure_constraint(submonoid_closure_contains(a))
} by {
    forall(x: M, y: M) {
        if submonoid_closure_contains(a, x) and submonoid_closure_contains(a, y) {
            forall(s: Submonoid[M]) {
                if set_subset_submonoid(a, s) {
                    submonoid_closure_contains_of_set_subset_raw(a, s, x)
                    s.contains(x)
                    submonoid_closure_contains_of_set_subset_raw(a, s, y)
                    s.contains(y)
                    submonoid_mul_mem(s, x, y)
                    s.contains(x * y)
                }
            }
            submonoid_closure_contains(a, x * y)
        }
    }
}

/// The submonoid generated by a set satisfies the submonoid laws.
theorem submonoid_generated_constraint[M: Monoid](a: Set[M]) {
    submonoid_constraint(submonoid_closure_contains(a))
} by {
    submonoid_closure_identity_constraint(a)
    submonoid_closure_closure_constraint(a)
}

/// The smallest submonoid containing a set.
let submonoid_closure[M: Monoid](a: Set[M]) -> result: Submonoid[M] satisfy {
    Submonoid.new(submonoid_closure_contains(a)) = Option.some(result)
} by {
    submonoid_generated_constraint(a)
}

/// Membership in the closure means membership in every submonoid containing the set.
theorem submonoid_closure_contains_eq[M: Monoid](a: Set[M], x: M) {
    submonoid_closure(a).contains(x) = submonoid_closure_contains(a, x)
} by {
    submonoid_closure(a).contains(x) = submonoid_closure_contains(a, x)
}

/// Every generator belongs to the submonoid closure.
theorem submonoid_subset_closure[M: Monoid](a: Set[M]) {
    set_subset_submonoid(a, submonoid_closure(a))
} by {
    forall(x: M) {
        if a.contains(x) {
            forall(s: Submonoid[M]) {
                if set_subset_submonoid(a, s) {
                    s.contains(x)
                }
            }
            submonoid_closure_contains(a, x)
            submonoid_closure_contains_eq(a, x)
            submonoid_closure(a).contains(x)
        }
    }
}

/// A generator is a member of the submonoid closure.
theorem submonoid_closure_contains_of_set_contains[M: Monoid](a: Set[M], x: M) {
    a.contains(x) implies submonoid_closure(a).contains(x)
} by {
    if a.contains(x) {
        submonoid_subset_closure(a)
        submonoid_closure(a).contains(x)
    }
}

/// The submonoid closure is contained in any submonoid containing the set.
theorem submonoid_closure_subset_of_set_subset[M: Monoid](a: Set[M], s: Submonoid[M]) {
    set_subset_submonoid(a, s) implies submonoid_subset(submonoid_closure(a), s)
} by {
    if set_subset_submonoid(a, s) {
        forall(x: M) {
            if submonoid_closure(a).contains(x) {
                submonoid_closure_contains_eq(a, x)
                submonoid_closure_contains(a, x)
                submonoid_closure_contains_of_set_subset_raw(a, s, x)
            }
        }
    }
}

/// The submonoid closure is the least submonoid containing the set.
theorem submonoid_closure_le_iff_set_subset[M: Monoid](a: Set[M], s: Submonoid[M]) {
    submonoid_subset(submonoid_closure(a), s) = set_subset_submonoid(a, s)
} by {
    if submonoid_subset(submonoid_closure(a), s) {
        submonoid_subset_closure(a)
        set_subset_submonoid_trans(a, submonoid_closure(a), s)
        set_subset_submonoid(a, s)
    }
    if set_subset_submonoid(a, s) {
        submonoid_closure_subset_of_set_subset(a, s)
        submonoid_subset(submonoid_closure(a), s)
    }
    submonoid_subset(submonoid_closure(a), s) = set_subset_submonoid(a, s)
}

/// The submonoid closure and underlying set maps form a set-containment adjunction.
theorem submonoid_closure_as_set_galois_connection[M: Monoid](a: Set[M], s: Submonoid[M]) {
    submonoid_closure(a).as_set.subset(s.as_set) = a.subset(s.as_set)
} by {
    submonoid_subset_as_set_eq(submonoid_closure(a), s)
    submonoid_closure_le_iff_set_subset(a, s)
    set_subset_submonoid_as_set_eq(a, s)
    submonoid_closure(a).as_set.subset(s.as_set) = a.subset(s.as_set)
}

/// The closure of the underlying set of a submonoid is the submonoid itself.
theorem submonoid_closure_as_set[M: Monoid](s: Submonoid[M]) {
    submonoid_closure(s.as_set) = s
} by {
    submonoid_as_set_subset_self(s)
    submonoid_closure_subset_of_set_subset(s.as_set, s)
    submonoid_subset(submonoid_closure(s.as_set), s)
    submonoid_subset_closure(s.as_set)
    forall(x: M) {
        if s.contains(x) {
            s.as_set.contains(x)
            submonoid_closure(s.as_set).contains(x)
        }
    }
    submonoid_subset(s, submonoid_closure(s.as_set))
    submonoid_subset_antisymm(submonoid_closure(s.as_set), s)
}

/// Submonoid closure is monotone with respect to set inclusion.
theorem submonoid_closure_mono[M: Monoid](a: Set[M], b: Set[M]) {
    a.subset(b) implies submonoid_subset(submonoid_closure(a), submonoid_closure(b))
} by {
    if a.subset(b) {
        submonoid_subset_closure(b)
        set_subset_submonoid_of_set_subset(a, b, submonoid_closure(b))
        set_subset_submonoid(a, submonoid_closure(b))
        submonoid_closure_subset_of_set_subset(a, submonoid_closure(b))
        submonoid_subset(submonoid_closure(a), submonoid_closure(b))
    }
}

/// Equal sets have equal submonoid closures.
theorem submonoid_closure_eq_of_set_eq[M: Monoid](a: Set[M], b: Set[M]) {
    a = b implies submonoid_closure(a) = submonoid_closure(b)
} by {
    if a = b {
        submonoid_closure(a) = submonoid_closure(b)
    }
}

/// Applying submonoid closure twice gives the same submonoid.
theorem submonoid_closure_idempotent[M: Monoid](a: Set[M]) {
    submonoid_closure(submonoid_closure(a).as_set) = submonoid_closure(a)
} by {
    submonoid_closure_as_set(submonoid_closure(a))
}

/// The set closure induced by submonoid generation.
define submonoid_set_closure[M: Monoid](a: Set[M]) -> Set[M] {
    submonoid_closure(a).as_set
}

/// The submonoid set closure is the underlying set of the generated submonoid.
theorem submonoid_set_closure_at[M: Monoid](a: Set[M]) {
    submonoid_set_closure(a) = submonoid_closure(a).as_set
}

/// The submonoid set closure contains the original set.
theorem submonoid_set_closure_extensive[M: Monoid](a: Set[M]) {
    a.subset(submonoid_set_closure(a))
} by {
    submonoid_set_closure_at(a)
    submonoid_subset_closure(a)
    set_subset_submonoid_as_set_eq(a, submonoid_closure(a))
    a.subset(submonoid_closure(a).as_set)
    a.subset(submonoid_set_closure(a))
}

/// The submonoid set closure preserves inclusion.
theorem submonoid_set_closure_mono[M: Monoid](a: Set[M], b: Set[M]) {
    a.subset(b) implies submonoid_set_closure(a).subset(submonoid_set_closure(b))
} by {
    if a.subset(b) {
        submonoid_closure_mono(a, b)
        submonoid_subset(submonoid_closure(a), submonoid_closure(b))
        submonoid_subset_as_set_eq(submonoid_closure(a), submonoid_closure(b))
        submonoid_closure(a).as_set.subset(submonoid_closure(b).as_set)
        submonoid_set_closure_at(a)
        submonoid_set_closure_at(b)
        submonoid_set_closure(a).subset(submonoid_set_closure(b))
    }
}

/// The submonoid set closure is unchanged after two applications.
theorem submonoid_set_closure_idempotent[M: Monoid](a: Set[M]) {
    submonoid_set_closure(submonoid_set_closure(a)) = submonoid_set_closure(a)
} by {
    submonoid_set_closure_at(a)
    submonoid_set_closure_at(submonoid_set_closure(a))
    submonoid_closure_idempotent(a)
    submonoid_set_closure(submonoid_set_closure(a)) = submonoid_set_closure(a)
}

/// Submonoid set closure is monotone as a set map.
theorem submonoid_set_closure_is_monotone[M: Monoid] {
    is_subset_monotone_map(submonoid_set_closure[M])
} by {
    forall(a: Set[M], b: Set[M]) {
        if a.subset(b) {
            submonoid_set_closure_mono(a, b)
            submonoid_set_closure(a).subset(submonoid_set_closure(b))
        }
    }
}

/// Submonoid set closure is extensive as a set map.
theorem submonoid_set_closure_is_extensive[M: Monoid] {
    is_set_extensive_map(submonoid_set_closure[M])
} by {
    forall(a: Set[M]) {
        submonoid_set_closure_extensive(a)
        a.subset(submonoid_set_closure(a))
    }
}

/// Submonoid set closure is idempotent as a set map.
theorem submonoid_set_closure_is_idempotent[M: Monoid] {
    is_set_idempotent_map(submonoid_set_closure[M])
} by {
    forall(a: Set[M]) {
        submonoid_set_closure_idempotent(a)
        submonoid_set_closure(submonoid_set_closure(a)) = submonoid_set_closure(a)
    }
}

/// Submonoid generation induces a closure operator on sets.
theorem submonoid_set_closure_operator[M: Monoid] {
    is_set_closure_operator(submonoid_set_closure[M])
} by {
    submonoid_set_closure_is_monotone[M]
    submonoid_set_closure_is_extensive[M]
    submonoid_set_closure_is_idempotent[M]
    is_set_closure_operator(submonoid_set_closure[M])
}

/// The closure of the empty set is the identity submonoid.
theorem submonoid_closure_empty[M: Monoid] {
    submonoid_closure(Set[M].empty_set) = identity_submonoid[M]
} by {
    set_subset_submonoid(Set[M].empty_set, identity_submonoid[M])
    submonoid_closure_subset_of_set_subset(Set[M].empty_set, identity_submonoid[M])
    submonoid_subset(submonoid_closure(Set[M].empty_set), identity_submonoid[M])
    identity_submonoid_subset(submonoid_closure(Set[M].empty_set))
    submonoid_subset_antisymm(submonoid_closure(Set[M].empty_set), identity_submonoid[M])
}

/// The closure of the universal set is the full submonoid.
theorem submonoid_closure_universal[M: Monoid] {
    submonoid_closure(Set[M].universal_set) = full_submonoid[M]
} by {
    submonoid_subset_full(submonoid_closure(Set[M].universal_set))
    forall(x: M) {
        if full_submonoid[M].contains(x) {
            Set[M].universal_set.contains(x)
            submonoid_closure_contains_of_set_contains(Set[M].universal_set, x)
            submonoid_closure(Set[M].universal_set).contains(x)
        }
    }
    submonoid_subset(full_submonoid[M], submonoid_closure(Set[M].universal_set))
    submonoid_subset_antisymm(submonoid_closure(Set[M].universal_set), full_submonoid[M])
}

/// The least submonoid containing two submonoids.
define submonoid_sup[M: Monoid](a: Submonoid[M], b: Submonoid[M]) -> Submonoid[M] {
    submonoid_closure(a.as_set.union(b.as_set))
}

/// The join of two submonoids is the closure of the union of their underlying sets.
theorem submonoid_sup_eq_closure_union[M: Monoid](a: Submonoid[M], b: Submonoid[M]) {
    submonoid_sup(a, b) = submonoid_closure(a.as_set.union(b.as_set))
}

/// The left submonoid is contained in the join.
theorem submonoid_subset_sup_left[M: Monoid](a: Submonoid[M], b: Submonoid[M]) {
    submonoid_subset(a, submonoid_sup(a, b))
} by {
    forall(x: M) {
        if a.contains(x) {
            submonoid_as_set_contains_eq(a, x)
            a.as_set.contains(x)
            union_contains_left(a.as_set, b.as_set, x)
            a.as_set.union(b.as_set).contains(x)
            submonoid_closure_contains_of_set_contains(a.as_set.union(b.as_set), x)
            submonoid_closure(a.as_set.union(b.as_set)).contains(x)
            submonoid_sup(a, b).contains(x)
        }
    }
}

/// The right submonoid is contained in the join.
theorem submonoid_subset_sup_right[M: Monoid](a: Submonoid[M], b: Submonoid[M]) {
    submonoid_subset(b, submonoid_sup(a, b))
} by {
    forall(x: M) {
        if b.contains(x) {
            submonoid_as_set_contains_eq(b, x)
            b.as_set.contains(x)
            union_contains_right(a.as_set, b.as_set, x)
            a.as_set.union(b.as_set).contains(x)
            submonoid_closure_contains_of_set_contains(a.as_set.union(b.as_set), x)
            submonoid_closure(a.as_set.union(b.as_set)).contains(x)
            submonoid_sup(a, b).contains(x)
        }
    }
}

/// The join is contained in every common upper bound.
theorem submonoid_sup_subset_of_subset_left_right[M: Monoid](
    a: Submonoid[M],
    b: Submonoid[M],
    c: Submonoid[M]
) {
    submonoid_subset(a, c) and submonoid_subset(b, c) implies submonoid_subset(submonoid_sup(a, b), c)
} by {
    if submonoid_subset(a, c) and submonoid_subset(b, c) {
        forall(x: M) {
            if a.as_set.union(b.as_set).contains(x) {
                union_contains_eq(a.as_set, b.as_set, x)
                if a.as_set.contains(x) {
                    submonoid_as_set_contains_eq(a, x)
                    a.contains(x)
                    c.contains(x)
                } else {
                    b.as_set.contains(x)
                    submonoid_as_set_contains_eq(b, x)
                    b.contains(x)
                    c.contains(x)
                }
            }
        }
        set_subset_submonoid(a.as_set.union(b.as_set), c)
        submonoid_closure_subset_of_set_subset(a.as_set.union(b.as_set), c)
        submonoid_subset(submonoid_closure(a.as_set.union(b.as_set)), c)
        submonoid_subset(submonoid_sup(a, b), c)
    }
}

/// Containment of a join is equivalent to containment of both submonoids.
theorem submonoid_sup_subset_iff[M: Monoid](
    a: Submonoid[M],
    b: Submonoid[M],
    c: Submonoid[M]
) {
    submonoid_subset(submonoid_sup(a, b), c) = (submonoid_subset(a, c) and submonoid_subset(b, c))
} by {
    if submonoid_subset(submonoid_sup(a, b), c) {
        submonoid_subset_sup_left(a, b)
        submonoid_subset_trans(a, submonoid_sup(a, b), c)
        submonoid_subset(a, c)
        submonoid_subset_sup_right(a, b)
        submonoid_subset_trans(b, submonoid_sup(a, b), c)
        submonoid_subset(b, c)
        submonoid_subset(a, c) and submonoid_subset(b, c)
    }
    if submonoid_subset(a, c) and submonoid_subset(b, c) {
        submonoid_sup_subset_of_subset_left_right(a, b, c)
        submonoid_subset(submonoid_sup(a, b), c)
    }
    submonoid_subset(submonoid_sup(a, b), c) = (submonoid_subset(a, c) and submonoid_subset(b, c))
}

attributes Submonoid[M: Monoid] {
    /// The smallest submonoid containing a set.
    let closure: Set[M] -> Submonoid[M] = submonoid_closure

    /// The least submonoid containing this submonoid and another submonoid.
    let sup: (Submonoid[M], Submonoid[M]) -> Submonoid[M] = submonoid_sup
}

/// True if a submonoid is generated by a finite set.
define submonoid_is_finitely_generated[M: Monoid](s: Submonoid[M]) -> Bool {
    exists(a: Set[M]) {
        a.is_finite and submonoid_closure(a) = s
    }
}

/// A finitely generated submonoid has a finite generating set.
theorem submonoid_is_finitely_generated_witness[M: Monoid](s: Submonoid[M]) {
    submonoid_is_finitely_generated(s) implies exists(a: Set[M]) {
        a.is_finite and submonoid_closure(a) = s
    }
}

/// A submonoid equal to the closure of a finite set is finitely generated.
theorem submonoid_is_finitely_generated_of_closure_eq[M: Monoid](a: Set[M], s: Submonoid[M]) {
    a.is_finite and submonoid_closure(a) = s implies submonoid_is_finitely_generated(s)
} by {
    if a.is_finite and submonoid_closure(a) = s {
        exists(b: Set[M]) {
            b.is_finite and submonoid_closure(b) = s
        }
        submonoid_is_finitely_generated(s)
    }
}

/// Equality preserves finite generation of submonoids.
theorem submonoid_is_finitely_generated_of_eq[M: Monoid](s: Submonoid[M], t: Submonoid[M]) {
    submonoid_is_finitely_generated(s) and s = t implies submonoid_is_finitely_generated(t)
} by {
    if submonoid_is_finitely_generated(s) and s = t {
        submonoid_is_finitely_generated_witness(s)
        let a: Set[M] satisfy {
            a.is_finite and submonoid_closure(a) = s
        }
        submonoid_closure(a) = t
        submonoid_is_finitely_generated_of_closure_eq(a, t)
        submonoid_is_finitely_generated(t)
    }
}

/// The closure of a finite set is a finitely generated submonoid.
theorem submonoid_closure_is_finitely_generated[M: Monoid](a: Set[M]) {
    a.is_finite implies submonoid_is_finitely_generated(submonoid_closure(a))
} by {
    if a.is_finite {
        submonoid_is_finitely_generated_of_closure_eq(a, submonoid_closure(a))
    }
}

/// The closure of one element is a finitely generated submonoid.
theorem submonoid_closure_singleton_is_finitely_generated[M: Monoid](a: M) {
    submonoid_is_finitely_generated(submonoid_closure(Set[M].singleton(a)))
} by {
    singleton_set_is_finite(a)
    submonoid_closure_is_finitely_generated(Set[M].singleton(a))
}

/// The generator belongs to the submonoid generated by it.
theorem submonoid_closure_singleton_contains[M: Monoid](a: M) {
    submonoid_closure(Set[M].singleton(a)).contains(a)
} by {
    singleton_contains_eq(a, a)
    submonoid_closure_contains_of_set_contains(Set[M].singleton(a), a)
}

/// The identity submonoid is finitely generated.
theorem identity_submonoid_is_finitely_generated[M: Monoid] {
    submonoid_is_finitely_generated(identity_submonoid[M])
} by {
    empty_set_is_finite[M]
    submonoid_closure_is_finitely_generated(Set[M].empty_set)
    submonoid_is_finitely_generated(submonoid_closure(Set[M].empty_set))
    submonoid_closure_empty[M]
    submonoid_is_finitely_generated(identity_submonoid[M])
}

/// The subsemigroup predicate associated to a submonoid.
theorem submonoid_subsemigroup_constraint[M: Monoid](s: Submonoid[M]) {
    subsemigroup_closure_constraint(s.contains)
} by {
    submonoid_closure_constraint(s.contains)
}

/// The subsemigroup associated to a submonoid.
let submonoid_to_subsemigroup[M: Monoid](s: Submonoid[M]) -> result: Subsemigroup[M] satisfy {
    Subsemigroup.new(s.contains) = Option.some(result)
} by {
    submonoid_subsemigroup_constraint(s)
}

/// The subsemigroup associated to a submonoid has the same membership predicate.
theorem submonoid_to_subsemigroup_contains[M: Monoid](s: Submonoid[M]) {
    submonoid_to_subsemigroup(s).contains = s.contains
}

/// Membership in the subsemigroup associated to a submonoid is membership in the submonoid.
theorem submonoid_to_subsemigroup_contains_eq[M: Monoid](s: Submonoid[M], x: M) {
    submonoid_to_subsemigroup(s).contains(x) = s.contains(x)
} by {
    submonoid_to_subsemigroup_contains(s)
    submonoid_to_subsemigroup(s).contains(x) = s.contains(x)
}

/// Membership in a submonoid is membership in its associated subsemigroup.
theorem submonoid_contains_to_subsemigroup_eq[M: Monoid](s: Submonoid[M], x: M) {
    s.contains(x) = submonoid_to_subsemigroup(s).contains(x)
} by {
    submonoid_to_subsemigroup_contains_eq(s, x)
    s.contains(x) = submonoid_to_subsemigroup(s).contains(x)
}

/// The associated subsemigroup has the same underlying set as the submonoid.
theorem submonoid_to_subsemigroup_as_set[M: Monoid](s: Submonoid[M]) {
    submonoid_to_subsemigroup(s).as_set = s.as_set
}

/// The inverse image membership predicate of a submonoid under a monoid homomorphism.
define submonoid_preimage_contains[M: Monoid, N: Monoid](
    f: MonoidHom[M, N],
    s: Submonoid[N],
    a: M
) -> Bool {
    s.contains(f.hom(a))
}

/// The inverse image of a submonoid contains the identity.
theorem submonoid_preimage_identity_constraint[M: Monoid, N: Monoid](
    f: MonoidHom[M, N],
    s: Submonoid[N]
) {
    submonoid_identity_constraint(submonoid_preimage_contains(f, s))
} by {
    monoid_hom_one(f)
    f.hom(M.1) = N.1
    submonoid_contains_identity(s)
    s.contains(N.1)
    s.contains(f.hom(M.1))
    submonoid_preimage_contains(f, s, M.1)
}

/// The inverse image of a submonoid is closed under multiplication.
theorem submonoid_preimage_closure_constraint[M: Monoid, N: Monoid](
    f: MonoidHom[M, N],
    s: Submonoid[N]
) {
    submonoid_closure_constraint(submonoid_preimage_contains(f, s))
} by {
    forall(a: M, b: M) {
        if submonoid_preimage_contains(f, s, a) and submonoid_preimage_contains(f, s, b) {
            s.contains(f.hom(a))
            s.contains(f.hom(b))
            submonoid_mul_mem(s, f.hom(a), f.hom(b))
            s.contains(f.hom(a) * f.hom(b))
            monoid_hom_mul(f, a, b)
            f.hom(a * b) = f.hom(a) * f.hom(b)
            s.contains(f.hom(a * b))
            submonoid_preimage_contains(f, s, a * b)
        }
    }
}

/// The inverse image of a submonoid is a submonoid.
theorem submonoid_preimage_constraint[M: Monoid, N: Monoid](
    f: MonoidHom[M, N],
    s: Submonoid[N]
) {
    submonoid_constraint(submonoid_preimage_contains(f, s))
} by {
    submonoid_preimage_identity_constraint(f, s)
    submonoid_preimage_closure_constraint(f, s)
}

/// The inverse image of a submonoid under a monoid homomorphism.
let submonoid_preimage[M: Monoid, N: Monoid](f: MonoidHom[M, N], s: Submonoid[N]) -> result: Submonoid[M] satisfy {
    Submonoid.new(submonoid_preimage_contains(f, s)) = Option.some(result)
} by {
    submonoid_preimage_constraint(f, s)
}

/// Membership in an inverse image means the mapped element belongs to the target submonoid.
theorem submonoid_preimage_contains_eq[M: Monoid, N: Monoid](
    f: MonoidHom[M, N],
    s: Submonoid[N],
    a: M
) {
    submonoid_preimage(f, s).contains(a) = s.contains(f.hom(a))
} by {
    submonoid_preimage(f, s).contains(a) = submonoid_preimage_contains(f, s, a)
    submonoid_preimage_contains(f, s, a) = s.contains(f.hom(a))
}

/// Membership in a preimage follows from membership of the mapped element.
theorem submonoid_preimage_contains_of_contains_map[M: Monoid, N: Monoid](
    f: MonoidHom[M, N],
    s: Submonoid[N],
    a: M
) {
    s.contains(f.hom(a)) implies submonoid_preimage(f, s).contains(a)
} by {
    if s.contains(f.hom(a)) {
        submonoid_preimage_contains_eq(f, s, a)
        submonoid_preimage(f, s).contains(a)
    }
}

/// Membership in the target follows from membership in the preimage.
theorem submonoid_contains_map_of_preimage_contains[M: Monoid, N: Monoid](
    f: MonoidHom[M, N],
    s: Submonoid[N],
    a: M
) {
    submonoid_preimage(f, s).contains(a) implies s.contains(f.hom(a))
} by {
    if submonoid_preimage(f, s).contains(a) {
        submonoid_preimage_contains_eq(f, s, a)
        s.contains(f.hom(a))
    }
}

/// Inverse images are monotone with respect to submonoid containment.
theorem submonoid_preimage_subset_of_subset[M: Monoid, N: Monoid](
    f: MonoidHom[M, N],
    s: Submonoid[N],
    t: Submonoid[N]
) {
    submonoid_subset(s, t) implies submonoid_subset(submonoid_preimage(f, s), submonoid_preimage(f, t))
} by {
    if submonoid_subset(s, t) {
        forall(a: M) {
            if submonoid_preimage(f, s).contains(a) {
                submonoid_preimage_contains_eq(f, s, a)
                s.contains(f.hom(a))
                submonoid_subset(s, t) = forall(x: N) {
                    s.contains(x) implies t.contains(x)
                }
                t.contains(f.hom(a))
                submonoid_preimage_contains_eq(f, t, a)
                submonoid_preimage(f, t).contains(a)
            }
        }
    }
}

/// The inverse image of an intersection is the intersection of the inverse images.
theorem submonoid_preimage_intersection[M: Monoid, N: Monoid](
    f: MonoidHom[M, N],
    s: Submonoid[N],
    t: Submonoid[N]
) {
    submonoid_preimage(f, s.intersection(t)) =
        submonoid_preimage(f, s).intersection(submonoid_preimage(f, t))
} by {
    forall(a: M) {
        submonoid_preimage_contains_eq(f, s.intersection(t), a)
        submonoid_preimage_contains_eq(f, s, a)
        submonoid_preimage_contains_eq(f, t, a)
        submonoid_intersection_contains_eq(s, t, f.hom(a))
        submonoid_intersection_contains_eq(submonoid_preimage(f, s), submonoid_preimage(f, t), a)
        submonoid_preimage(f, s.intersection(t)).contains(a) =
            submonoid_preimage(f, s).intersection(submonoid_preimage(f, t)).contains(a)
    }
    submonoid_ext(submonoid_preimage(f, s.intersection(t)),
        submonoid_preimage(f, s).intersection(submonoid_preimage(f, t)))
}

/// The inverse image of the full submonoid is the full submonoid.
theorem submonoid_preimage_full[M: Monoid, N: Monoid](f: MonoidHom[M, N]) {
    submonoid_preimage(f, full_submonoid[N]) = full_submonoid[M]
} by {
    submonoid_subset_full(submonoid_preimage(f, full_submonoid[N]))
    forall(a: M) {
        if full_submonoid[M].contains(a) {
            full_submonoid_contains_everything(f.hom(a))
            submonoid_preimage_contains_eq(f, full_submonoid[N], a)
            submonoid_preimage(f, full_submonoid[N]).contains(a)
        }
    }
    submonoid_subset(full_submonoid[M], submonoid_preimage(f, full_submonoid[N]))
    submonoid_subset_antisymm(submonoid_preimage(f, full_submonoid[N]), full_submonoid[M])
}

/// The direct image membership predicate of a submonoid under a monoid homomorphism.
define submonoid_image_contains[M: Monoid, N: Monoid](
    f: MonoidHom[M, N],
    s: Submonoid[M]
) -> N -> Bool {
    function(y: N) {
        exists(x: M) {
            s.contains(x) and f.hom(x) = y
        }
    }
}

/// The direct image of a submonoid contains the identity.
theorem submonoid_image_identity_constraint[M: Monoid, N: Monoid](
    f: MonoidHom[M, N],
    s: Submonoid[M]
) {
    submonoid_identity_constraint(submonoid_image_contains(f, s))
} by {
    submonoid_contains_identity(s)
    monoid_hom_one(f)
    s.contains(M.1)
    f.hom(M.1) = N.1
    s.contains(M.1) and f.hom(M.1) = N.1
    exists(x: M) {
        s.contains(x) and f.hom(x) = N.1
    }
    submonoid_image_contains(f, s)(N.1)
}

/// The product of two direct-image members belongs to the direct image.
theorem submonoid_image_mul_mem[M: Monoid, N: Monoid](
    f: MonoidHom[M, N],
    s: Submonoid[M],
    a: N,
    b: N
) {
    submonoid_image_contains(f, s)(a) and submonoid_image_contains(f, s)(b)
        implies submonoid_image_contains(f, s)(a * b)
} by {
    if submonoid_image_contains(f, s)(a) and submonoid_image_contains(f, s)(b) {
        let xa: M satisfy {
            s.contains(xa) and f.hom(xa) = a
        }
        let xb: M satisfy {
            s.contains(xb) and f.hom(xb) = b
        }
        submonoid_mul_mem(s, xa, xb)
        s.contains(xa * xb)
        monoid_hom_mul(f, xa, xb)
        f.hom(xa * xb) = f.hom(xa) * f.hom(xb)
        f.hom(xa) * f.hom(xb) = a * b
        f.hom(xa * xb) = a * b
        s.contains(xa * xb) and f.hom(xa * xb) = a * b
        exists(x: M) {
            s.contains(x) and f.hom(x) = a * b
        }
        submonoid_image_contains(f, s)(a * b)
    }
}

/// The direct image of a submonoid is closed under multiplication.
theorem submonoid_image_closure_constraint[M: Monoid, N: Monoid](
    f: MonoidHom[M, N],
    s: Submonoid[M]
) {
    submonoid_closure_constraint(submonoid_image_contains(f, s))
} by {
    let contains: N -> Bool = submonoid_image_contains(f, s)
    forall(y: N) {
        contains(y) = exists(x: M) {
            s.contains(x) and f.hom(x) = y
        }
    }
    forall(a: N, b: N) {
        if contains(a) and contains(b) {
            let xa: M satisfy {
                s.contains(xa) and f.hom(xa) = a
            }
            let xb: M satisfy {
                s.contains(xb) and f.hom(xb) = b
            }
            submonoid_mul_mem(s, xa, xb)
            s.contains(xa * xb)
            monoid_hom_mul(f, xa, xb)
            f.hom(xa * xb) = f.hom(xa) * f.hom(xb)
            f.hom(xa) * f.hom(xb) = a * b
            f.hom(xa * xb) = a * b
            s.contains(xa * xb) and f.hom(xa * xb) = a * b
            exists(x: M) {
                s.contains(x) and f.hom(x) = a * b
            }
            contains(a * b)
        }
    }
    submonoid_closure_constraint(contains)
    submonoid_closure_constraint(submonoid_image_contains(f, s))
}

/// The direct image of a submonoid is a submonoid.
theorem submonoid_image_constraint[M: Monoid, N: Monoid](
    f: MonoidHom[M, N],
    s: Submonoid[M]
) {
    submonoid_constraint(submonoid_image_contains(f, s))
} by {
    submonoid_image_identity_constraint(f, s)
    submonoid_image_closure_constraint(f, s)
}

/// The direct image of a submonoid under a monoid homomorphism.
let submonoid_image[M: Monoid, N: Monoid](f: MonoidHom[M, N], s: Submonoid[M]) -> result: Submonoid[N] satisfy {
    Submonoid.new(submonoid_image_contains(f, s)) = Option.some(result)
} by {
    submonoid_image_constraint(f, s)
}

/// Membership in a direct image means being the image of a source member.
theorem submonoid_image_contains_eq[M: Monoid, N: Monoid](
    f: MonoidHom[M, N],
    s: Submonoid[M],
    y: N
) {
    submonoid_image(f, s).contains(y) = exists(x: M) {
        s.contains(x) and f.hom(x) = y
    }
} by {
    submonoid_image(f, s).contains(y) = submonoid_image_contains(f, s)(y)
    submonoid_image_contains(f, s)(y) = exists(x: M) {
        s.contains(x) and f.hom(x) = y
    }
}

/// The image of a member of a submonoid belongs to the direct image.
theorem submonoid_image_contains_map[M: Monoid, N: Monoid](
    f: MonoidHom[M, N],
    s: Submonoid[M],
    x: M
) {
    s.contains(x) implies submonoid_image(f, s).contains(f.hom(x))
} by {
    if s.contains(x) {
        s.contains(x) and f.hom(x) = f.hom(x)
        exists(a: M) {
            s.contains(a) and f.hom(a) = f.hom(x)
        }
        submonoid_image_contains_eq(f, s, f.hom(x))
        submonoid_image(f, s).contains(f.hom(x))
    }
}

/// Direct images are monotone with respect to submonoid containment.
theorem submonoid_image_subset_of_subset[M: Monoid, N: Monoid](
    f: MonoidHom[M, N],
    s: Submonoid[M],
    t: Submonoid[M]
) {
    submonoid_subset(s, t) implies submonoid_subset(submonoid_image(f, s), submonoid_image(f, t))
} by {
    if submonoid_subset(s, t) {
        submonoid_subset(s, t) = forall(x: M) {
            s.contains(x) implies t.contains(x)
        }
        forall(y: N) {
            if submonoid_image(f, s).contains(y) {
                submonoid_image_contains_eq(f, s, y)
                let x: M satisfy {
                    s.contains(x) and f.hom(x) = y
                }
                t.contains(x)
                submonoid_image_contains_map(f, t, x)
                submonoid_image(f, t).contains(f.hom(x))
                submonoid_image(f, t).contains(y)
            }
        }
    }
}

/// The image of the inverse image of a submonoid is contained in that submonoid.
theorem submonoid_image_preimage_contains[M: Monoid, N: Monoid](
    f: MonoidHom[M, N],
    t: Submonoid[N],
    y: N
) {
    submonoid_image(f, submonoid_preimage(f, t)).contains(y) implies t.contains(y)
} by {
    if submonoid_image(f, submonoid_preimage(f, t)).contains(y) {
        submonoid_image_contains_eq(f, submonoid_preimage(f, t), y)
        let x: M satisfy {
            submonoid_preimage(f, t).contains(x) and f.hom(x) = y
        }
        submonoid_preimage_contains_eq(f, t, x)
        t.contains(f.hom(x))
        t.contains(y)
    }
}

/// The image of an inverse image is contained in the original submonoid.
theorem submonoid_image_preimage_subset[M: Monoid, N: Monoid](
    f: MonoidHom[M, N],
    t: Submonoid[N]
) {
    submonoid_subset(submonoid_image(f, submonoid_preimage(f, t)), t)
} by {
    let image = submonoid_image(f, submonoid_preimage(f, t))
    forall(y: N) {
        if image.contains(y) {
            submonoid_image_preimage_contains(f, t, y)
            t.contains(y)
        }
    }
    submonoid_subset(image, t) = forall(y: N) {
        image.contains(y) implies t.contains(y)
    }
    submonoid_subset(image, t)
}

/// A submonoid is contained in the inverse image of its direct image.
theorem submonoid_preimage_image_contains[M: Monoid, N: Monoid](
    f: MonoidHom[M, N],
    s: Submonoid[M],
    x: M
) {
    s.contains(x) implies submonoid_preimage(f, submonoid_image(f, s)).contains(x)
} by {
    if s.contains(x) {
        submonoid_image_contains_map(f, s, x)
        submonoid_image(f, s).contains(f.hom(x))
        submonoid_preimage_contains_eq(f, submonoid_image(f, s), x)
        submonoid_preimage(f, submonoid_image(f, s)).contains(x)
    }
}

/// A submonoid is contained in the inverse image of its direct image.
theorem submonoid_subset_preimage_image[M: Monoid, N: Monoid](
    f: MonoidHom[M, N],
    s: Submonoid[M]
) {
    submonoid_subset(s, submonoid_preimage(f, submonoid_image(f, s)))
} by {
    let preimage = submonoid_preimage(f, submonoid_image(f, s))
    forall(x: M) {
        if s.contains(x) {
            submonoid_preimage_image_contains(f, s, x)
            preimage.contains(x)
        }
    }
    submonoid_subset(s, preimage) = forall(x: M) {
        s.contains(x) implies preimage.contains(x)
    }
    submonoid_subset(s, preimage)
}

/// If a submonoid is contained in a preimage, then its image is contained in the target.
theorem submonoid_image_subset_of_subset_preimage[M: Monoid, N: Monoid](
    f: MonoidHom[M, N],
    s: Submonoid[M],
    t: Submonoid[N]
) {
    submonoid_subset(s, submonoid_preimage(f, t)) implies submonoid_subset(submonoid_image(f, s), t)
} by {
    if submonoid_subset(s, submonoid_preimage(f, t)) {
        submonoid_image_subset_of_subset(f, s, submonoid_preimage(f, t))
        submonoid_subset(submonoid_image(f, s), submonoid_image(f, submonoid_preimage(f, t)))
        submonoid_image_preimage_subset(f, t)
        submonoid_subset(submonoid_image(f, submonoid_preimage(f, t)), t)
        submonoid_subset_trans(submonoid_image(f, s), submonoid_image(f, submonoid_preimage(f, t)), t)
    }
}

/// If an image is contained in a submonoid, then the source is contained in the preimage.
theorem submonoid_subset_preimage_of_image_subset[M: Monoid, N: Monoid](
    f: MonoidHom[M, N],
    s: Submonoid[M],
    t: Submonoid[N]
) {
    submonoid_subset(submonoid_image(f, s), t) implies submonoid_subset(s, submonoid_preimage(f, t))
} by {
    if submonoid_subset(submonoid_image(f, s), t) {
        submonoid_subset_preimage_image(f, s)
        submonoid_preimage_subset_of_subset(f, submonoid_image(f, s), t)
        submonoid_subset(submonoid_preimage(f, submonoid_image(f, s)), submonoid_preimage(f, t))
        submonoid_subset_trans(s, submonoid_preimage(f, submonoid_image(f, s)), submonoid_preimage(f, t))
    }
}

/// Direct image and inverse image form a Galois connection on submonoids.
theorem submonoid_image_subset_iff_subset_preimage[M: Monoid, N: Monoid](
    f: MonoidHom[M, N],
    s: Submonoid[M],
    t: Submonoid[N]
) {
    submonoid_subset(submonoid_image(f, s), t) = submonoid_subset(s, submonoid_preimage(f, t))
} by {
    if submonoid_subset(submonoid_image(f, s), t) {
        submonoid_subset_preimage_of_image_subset(f, s, t)
        submonoid_subset(s, submonoid_preimage(f, t))
    }
    if submonoid_subset(s, submonoid_preimage(f, t)) {
        submonoid_image_subset_of_subset_preimage(f, s, t)
        submonoid_subset(submonoid_image(f, s), t)
    }
    submonoid_subset(submonoid_image(f, s), t) = submonoid_subset(s, submonoid_preimage(f, t))
}

/// The kernel of a monoid homomorphism as a submonoid.
define monoid_hom_kernel[M: Monoid, N: Monoid](f: MonoidHom[M, N]) -> Submonoid[M] {
    submonoid_preimage(f, identity_submonoid[N])
}

/// Membership in the kernel means the homomorphism maps the element to the identity.
theorem monoid_hom_kernel_contains_eq[M: Monoid, N: Monoid](f: MonoidHom[M, N], a: M) {
    monoid_hom_kernel(f).contains(a) = (f.hom(a) = N.1)
} by {
    monoid_hom_kernel(f) = submonoid_preimage(f, identity_submonoid[N])
    submonoid_preimage_contains_eq(f, identity_submonoid[N], a)
    if monoid_hom_kernel(f).contains(a) {
        identity_submonoid[N].contains(f.hom(a))
        identity_submonoid_only_has_identity(f.hom(a))
        f.hom(a) = N.1
    }
    if f.hom(a) = N.1 {
        submonoid_contains_identity(identity_submonoid[N])
        identity_submonoid[N].contains(N.1)
        identity_submonoid[N].contains(f.hom(a))
        monoid_hom_kernel(f).contains(a)
    }
    monoid_hom_kernel(f).contains(a) = (f.hom(a) = N.1)
}

/// An element belongs to the kernel when its image is the identity.
theorem monoid_hom_kernel_contains_of_maps_to_identity[M: Monoid, N: Monoid](f: MonoidHom[M, N], a: M) {
    f.hom(a) = N.1 implies monoid_hom_kernel(f).contains(a)
} by {
    if f.hom(a) = N.1 {
        monoid_hom_kernel_contains_eq(f, a)
        monoid_hom_kernel(f).contains(a)
    }
}

/// A kernel element maps to the identity.
theorem monoid_hom_maps_to_identity_of_kernel_contains[M: Monoid, N: Monoid](f: MonoidHom[M, N], a: M) {
    monoid_hom_kernel(f).contains(a) implies f.hom(a) = N.1
} by {
    if monoid_hom_kernel(f).contains(a) {
        monoid_hom_kernel_contains_eq(f, a)
        f.hom(a) = N.1
    }
}


/// Submonoid join is commutative.
theorem submonoid_sup_comm[M: Monoid](a: Submonoid[M], b: Submonoid[M]) {
    submonoid_sup(a, b) = submonoid_sup(b, a)
} by {
    submonoid_subset_sup_right(b, a)
    submonoid_subset_sup_left(b, a)
    submonoid_sup_subset_of_subset_left_right(a, b, submonoid_sup(b, a))
    submonoid_subset_sup_right(a, b)
    submonoid_subset_sup_left(a, b)
    submonoid_sup_subset_of_subset_left_right(b, a, submonoid_sup(a, b))
    submonoid_subset_antisymm(submonoid_sup(a, b), submonoid_sup(b, a))
}

/// Joining a submonoid with itself gives the same submonoid.
theorem submonoid_sup_idempotent[M: Monoid](a: Submonoid[M]) {
    submonoid_sup(a, a) = a
} by {
    submonoid_subset_refl(a)
    submonoid_sup_subset_of_subset_left_right(a, a, a)
    submonoid_subset_sup_left(a, a)
    submonoid_subset_antisymm(submonoid_sup(a, a), a)
}

/// Submonoid join is associative.
theorem submonoid_sup_assoc[M: Monoid](
    a: Submonoid[M],
    b: Submonoid[M],
    c: Submonoid[M]
) {
    submonoid_sup(submonoid_sup(a, b), c) = submonoid_sup(a, submonoid_sup(b, c))
} by {
    submonoid_subset_sup_left(a, submonoid_sup(b, c))
    submonoid_subset_sup_right(a, submonoid_sup(b, c))
    submonoid_subset_sup_left(b, c)
    submonoid_subset_sup_right(b, c)
    submonoid_subset_trans(b, submonoid_sup(b, c), submonoid_sup(a, submonoid_sup(b, c)))
    submonoid_subset_trans(c, submonoid_sup(b, c), submonoid_sup(a, submonoid_sup(b, c)))
    submonoid_sup_subset_of_subset_left_right(a, b, submonoid_sup(a, submonoid_sup(b, c)))
    submonoid_sup_subset_of_subset_left_right(submonoid_sup(a, b), c, submonoid_sup(a, submonoid_sup(b, c)))
    submonoid_subset_sup_left(submonoid_sup(a, b), c)
    submonoid_subset_sup_right(submonoid_sup(a, b), c)
    submonoid_subset_sup_left(a, b)
    submonoid_subset_sup_right(a, b)
    submonoid_subset_trans(a, submonoid_sup(a, b), submonoid_sup(submonoid_sup(a, b), c))
    submonoid_subset_trans(b, submonoid_sup(a, b), submonoid_sup(submonoid_sup(a, b), c))
    submonoid_sup_subset_of_subset_left_right(b, c, submonoid_sup(submonoid_sup(a, b), c))
    submonoid_sup_subset_of_subset_left_right(a, submonoid_sup(b, c), submonoid_sup(submonoid_sup(a, b), c))
    submonoid_subset_antisymm(submonoid_sup(submonoid_sup(a, b), c), submonoid_sup(a, submonoid_sup(b, c)))
}
