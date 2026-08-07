from algebra.monoid.monoid import Monoid, MonoidHom, compose_monoid_hom, compose_monoid_hom_hom
from data.basic.functions import compose
from data.basic.set import Set, set_image, set_preimage, maps_into_set_image, set_image_contains_witness,
    set_image_subset_iff_subset_preimage, set_image_compose, set_preimage_compose,
    set_image_preimage_closure, set_image_preimage_closure_extensive,
    set_image_preimage_closure_monotone, set_image_preimage_closure_idempotent,
    set_preimage_image_kernel, set_preimage_image_kernel_monotone,
    set_preimage_image_kernel_reductive, set_preimage_image_kernel_idempotent,
    is_subset_monotone_map, is_set_extensive_map, is_set_reductive_map,
    is_set_idempotent_map, is_set_closure_operator, is_set_kernel_operator

/// The image of a set under a monoid homomorphism.
define monoid_hom_image[M: Monoid, N: Monoid](f: MonoidHom[M, N], s: Set[M]) -> Set[N] {
    set_image(s, f.hom)
}

/// The preimage of a set under a monoid homomorphism.
define monoid_hom_preimage[M: Monoid, N: Monoid](f: MonoidHom[M, N], s: Set[N]) -> Set[M] {
    set_preimage(f.hom, s)
}

/// The source closure induced by image followed by preimage along a monoid homomorphism.
define monoid_hom_image_preimage_closure[M: Monoid, N: Monoid](
    f: MonoidHom[M, N],
    s: Set[M]
) -> Set[M] {
    set_image_preimage_closure(f.hom, s)
}

/// The target kernel induced by preimage followed by image along a monoid homomorphism.
define monoid_hom_preimage_image_kernel[M: Monoid, N: Monoid](
    f: MonoidHom[M, N],
    s: Set[N]
) -> Set[N] {
    set_preimage_image_kernel(f.hom, s)
}

/// Membership in the image of a monoid homomorphism has a source witness.
theorem monoid_hom_image_contains_witness[M: Monoid, N: Monoid](
    f: MonoidHom[M, N],
    s: Set[M],
    y: N
) {
    monoid_hom_image(f, s).contains(y) implies exists(x: M) {
        s.contains(x) and y = f.hom(x)
    }
} by {
    set_image_contains_witness(s, f.hom, y)
}

/// A member of a set maps into its image under a monoid homomorphism.
theorem monoid_hom_maps_into_image[M: Monoid, N: Monoid](
    f: MonoidHom[M, N],
    s: Set[M],
    x: M
) {
    s.contains(x) implies monoid_hom_image(f, s).contains(f.hom(x))
} by {
    if s.contains(x) {
        maps_into_set_image(s, f.hom, x)
        monoid_hom_image(f, s) = set_image(s, f.hom)
    }
}

/// Image containment is equivalent to source containment in the preimage.
theorem monoid_hom_image_subset_iff_subset_preimage[M: Monoid, N: Monoid](
    f: MonoidHom[M, N],
    s: Set[M],
    t: Set[N]
) {
    monoid_hom_image(f, s).subset(t) = s.subset(monoid_hom_preimage(f, t))
} by {
    set_image_subset_iff_subset_preimage(s, t, f.hom)
}

/// Image under a composite monoid homomorphism is iterated image.
theorem monoid_hom_image_compose[M: Monoid, N: Monoid, P: Monoid](
    f: MonoidHom[N, P],
    g: MonoidHom[M, N],
    s: Set[M]
) {
    monoid_hom_image(compose_monoid_hom(f, g), s) =
        monoid_hom_image(f, monoid_hom_image(g, s))
} by {
    compose_monoid_hom_hom(f, g)
    set_image_compose(s, g.hom, f.hom)
}

/// Preimage under a composite monoid homomorphism is iterated preimage.
theorem monoid_hom_preimage_compose[M: Monoid, N: Monoid, P: Monoid](
    f: MonoidHom[N, P],
    g: MonoidHom[M, N],
    s: Set[P]
) {
    monoid_hom_preimage(compose_monoid_hom(f, g), s) =
        monoid_hom_preimage(g, monoid_hom_preimage(f, s))
} by {
    compose_monoid_hom_hom(f, g)
    set_preimage_compose(f.hom, g.hom, s)
}

/// The source closure induced by a monoid homomorphism contains the original set.
theorem monoid_hom_image_preimage_closure_extensive[M: Monoid, N: Monoid](
    f: MonoidHom[M, N],
    s: Set[M]
) {
    s.subset(monoid_hom_image_preimage_closure(f, s))
} by {
    set_image_preimage_closure_extensive(f.hom, s)
}

/// The source closure induced by a monoid homomorphism preserves inclusion.
theorem monoid_hom_image_preimage_closure_monotone[M: Monoid, N: Monoid](
    f: MonoidHom[M, N]
) {
    is_subset_monotone_map(monoid_hom_image_preimage_closure(f))
} by {
    forall(s: Set[M], t: Set[M]) {
        if s.subset(t) {
            set_image_preimage_closure_monotone(f.hom, s, t)
            monoid_hom_image_preimage_closure(f, s).subset(
                monoid_hom_image_preimage_closure(f, t)
            )
        }
    }
}

/// The source closure induced by a monoid homomorphism is idempotent.
theorem monoid_hom_image_preimage_closure_idempotent[M: Monoid, N: Monoid](
    f: MonoidHom[M, N],
    s: Set[M]
) {
    monoid_hom_image_preimage_closure(f, monoid_hom_image_preimage_closure(f, s)) =
        monoid_hom_image_preimage_closure(f, s)
} by {
    set_image_preimage_closure_idempotent(f.hom, s)
}

/// The source closure induced by a monoid homomorphism is a set closure operator.
theorem monoid_hom_image_preimage_closure_operator[M: Monoid, N: Monoid](
    f: MonoidHom[M, N]
) {
    is_set_closure_operator(monoid_hom_image_preimage_closure(f))
} by {
    monoid_hom_image_preimage_closure_monotone(f)
    forall(s: Set[M]) {
        monoid_hom_image_preimage_closure_extensive(f, s)
        s.subset(monoid_hom_image_preimage_closure(f, s))
    }
    forall(s: Set[M]) {
        monoid_hom_image_preimage_closure_idempotent(f, s)
        monoid_hom_image_preimage_closure(f, monoid_hom_image_preimage_closure(f, s)) =
            monoid_hom_image_preimage_closure(f, s)
    }
}

/// The target kernel induced by a monoid homomorphism lies in the original set.
theorem monoid_hom_preimage_image_kernel_reductive[M: Monoid, N: Monoid](
    f: MonoidHom[M, N],
    s: Set[N]
) {
    monoid_hom_preimage_image_kernel(f, s).subset(s)
} by {
    set_preimage_image_kernel_reductive(f.hom, s)
}

/// The target kernel induced by a monoid homomorphism preserves inclusion.
theorem monoid_hom_preimage_image_kernel_monotone[M: Monoid, N: Monoid](
    f: MonoidHom[M, N]
) {
    is_subset_monotone_map(monoid_hom_preimage_image_kernel(f))
} by {
    forall(s: Set[N], t: Set[N]) {
        if s.subset(t) {
            set_preimage_image_kernel_monotone(f.hom, s, t)
            monoid_hom_preimage_image_kernel(f, s).subset(
                monoid_hom_preimage_image_kernel(f, t)
            )
        }
    }
}

/// The target kernel induced by a monoid homomorphism is idempotent.
theorem monoid_hom_preimage_image_kernel_idempotent[M: Monoid, N: Monoid](
    f: MonoidHom[M, N],
    s: Set[N]
) {
    monoid_hom_preimage_image_kernel(f, monoid_hom_preimage_image_kernel(f, s)) =
        monoid_hom_preimage_image_kernel(f, s)
} by {
    set_preimage_image_kernel_idempotent(f.hom, s)
}

/// The target kernel induced by a monoid homomorphism is a set kernel operator.
theorem monoid_hom_preimage_image_kernel_operator[M: Monoid, N: Monoid](
    f: MonoidHom[M, N]
) {
    is_set_kernel_operator(monoid_hom_preimage_image_kernel(f))
} by {
    monoid_hom_preimage_image_kernel_monotone(f)
    forall(s: Set[N]) {
        monoid_hom_preimage_image_kernel_reductive(f, s)
        monoid_hom_preimage_image_kernel(f, s).subset(s)
    }
    is_set_reductive_map(monoid_hom_preimage_image_kernel(f))
    forall(s: Set[N]) {
        monoid_hom_preimage_image_kernel_idempotent(f, s)
        monoid_hom_preimage_image_kernel(f, monoid_hom_preimage_image_kernel(f, s)) =
            monoid_hom_preimage_image_kernel(f, s)
    }
    is_set_idempotent_map(monoid_hom_preimage_image_kernel(f))
}
