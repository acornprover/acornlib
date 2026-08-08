from algebra.semigroup import Semigroup
from algebra.one import One
from data.basic.functions import identity_fn, compose, compose_assoc, compose_identity_left, compose_identity_right,
    function_eq_transport_predicate, function_eq_transport_predicate_rev

/// A multiplicative monoid is a multiplicative semigroup with an identity element.
/// Note that you need the identity to work on the right side as well as the left.
/// One doesn't imply the other.
typeclass M: Monoid extends Semigroup, One {
    /// The identity element must satisfy the identity property on the right.
    mul_identity_right(a: M) {
        a * M.1 = a
    }

    /// The identity element must satisfy the identity property on the left.
    mul_identity_left(a: M) {
        M.1 * a = a
    }
}

/// True if a function between monoids preserves the operation and the identity.
define is_monoid_hom[M: Monoid, N: Monoid](f: M -> N) -> Bool {
    f(M.1) = N.1 and forall(a: M, b: M) {
        f(a * b) = f(a) * f(b)
    }
}

/// The trivial monoid homomorphism maps every element of M to the identity element of N.
let trivial_monoid_hom[M: Monoid, N: Monoid]: M -> N = function(a: M) {
    N.1
}

/// The trivial monoid homomorphism preserves the operation and the identity.
theorem trivial_monoid_hom_is_hom[M: Monoid, N: Monoid] {
    is_monoid_hom(trivial_monoid_hom[M, N])
}

/// A monoid homomorphism that preserves the operation and the identity.
structure MonoidHom[M: Monoid, N: Monoid] {
    /// The mapping for the homomorphism.
    hom: M -> N
} constraint {
    is_monoid_hom(hom)
}

/// Monoid homomorphism extensionality: two homomorphisms are equal when they agree on every input.
theorem monoid_hom_ext[M: Monoid, N: Monoid](f: MonoidHom[M, N], g: MonoidHom[M, N]) {
    (forall(a: M) { f.hom(a) = g.hom(a) }) implies f = g
} by {
    if forall(a: M) { f.hom(a) = g.hom(a) } {
        f.hom = g.hom
    }
}

attributes MonoidHom[M: Monoid, N: Monoid] {
    /// Monoid homomorphism extensionality from pointwise equality of the homomorphism.
    let ext = monoid_hom_ext[M, N]
}

/// Equal monoid homomorphisms have equal underlying functions.
theorem monoid_hom_eq_hom[M: Monoid, N: Monoid](f: MonoidHom[M, N], g: MonoidHom[M, N]) {
    f = g implies f.hom = g.hom
}

/// Equal monoid homomorphisms have equal values at every element.
theorem monoid_hom_eq_apply[M: Monoid, N: Monoid](f: MonoidHom[M, N], g: MonoidHom[M, N], a: M) {
    f = g implies f.hom(a) = g.hom(a)
} by {
    if f = g {
        f.hom(a) = g.hom(a)
    }
}

/// Equality of monoid homomorphisms transports predicates on monoid homomorphisms.
theorem monoid_hom_eq_transport_predicate[M: Monoid, N: Monoid](
    p: MonoidHom[M, N] -> Bool,
    f: MonoidHom[M, N],
    g: MonoidHom[M, N]
) {
    f = g and p(f) implies p(g)
}

/// Equality of monoid homomorphisms transports predicates on monoid homomorphisms in the reverse direction.
theorem monoid_hom_eq_transport_predicate_rev[M: Monoid, N: Monoid](
    p: MonoidHom[M, N] -> Bool,
    f: MonoidHom[M, N],
    g: MonoidHom[M, N]
) {
    f = g and p(g) implies p(f)
}

/// Equality of underlying functions determines equality of monoid homomorphisms.
theorem monoid_hom_eq_of_hom_eq[M: Monoid, N: Monoid](f: MonoidHom[M, N], g: MonoidHom[M, N]) {
    f.hom = g.hom implies f = g
} by {
    if f.hom = g.hom {
        forall(a: M) {
            f.hom(a) = g.hom(a)
        }
        monoid_hom_ext(f, g)
    }
}

/// Pointwise equality determines equality of monoid homomorphisms.
theorem monoid_hom_eq_of_apply_eq[M: Monoid, N: Monoid](f: MonoidHom[M, N], g: MonoidHom[M, N]) {
    (forall(a: M) { f.hom(a) = g.hom(a) }) implies f = g
} by {
    if forall(a: M) { f.hom(a) = g.hom(a) } {
        monoid_hom_ext(f, g)
    }
}

/// A monoid homomorphism preserves the identity element.
theorem monoid_hom_one[M: Monoid, N: Monoid](f: MonoidHom[M, N]) {
    f.hom(M.1) = N.1
}

/// A monoid homomorphism preserves the operation.
theorem monoid_hom_mul[M: Monoid, N: Monoid](f: MonoidHom[M, N], a: M, b: M) {
    f.hom(a * b) = f.hom(a) * f.hom(b)
} by {
    f.hom(M.1) = N.1 and forall(x: M, y: M) {
        f.hom(x * y) = f.hom(x) * f.hom(y)
    }
}

/// Equality of functions transports the property of being a monoid homomorphism.
theorem function_eq_transport_monoid_hom[M: Monoid, N: Monoid](f: M -> N, g: M -> N) {
    f = g and is_monoid_hom(f) implies is_monoid_hom(g)
} by {
    if f = g and is_monoid_hom(f) {
        function_eq_transport_predicate(is_monoid_hom[M, N], f, g)
    }
}

/// Equality of functions transports the property of being a monoid homomorphism in the reverse direction.
theorem function_eq_transport_monoid_hom_rev[M: Monoid, N: Monoid](f: M -> N, g: M -> N) {
    f = g and is_monoid_hom(g) implies is_monoid_hom(f)
} by {
    if f = g and is_monoid_hom(g) {
        function_eq_transport_predicate_rev(is_monoid_hom[M, N], f, g)
    }
}

/// The identity function on a monoid preserves the monoid structure.
theorem identity_fn_is_monoid_hom[M: Monoid] {
    is_monoid_hom(identity_fn[M])
} by {
    forall(a: M, b: M) {
        identity_fn[M](a * b) = identity_fn[M](a) * identity_fn[M](b)
    }
}

/// The composition of two monoid homomorphisms preserves the monoid structure.
theorem compose_is_monoid_hom[M: Monoid, N: Monoid, P: Monoid](f: N -> P, g: M -> N) {
    is_monoid_hom(f) and is_monoid_hom(g) implies is_monoid_hom(compose(f, g))
} by {
    if is_monoid_hom(f) and is_monoid_hom(g) {
        is_monoid_hom(f) = (f(N.1) = P.1 and forall(x: N, y: N) {
            f(x * y) = f(x) * f(y)
        })
        is_monoid_hom(g) = (g(M.1) = N.1 and forall(x: M, y: M) {
            g(x * y) = g(x) * g(y)
        })
        compose(f, g)(M.1) = P.1
        forall(a: M, b: M) {
            f(g(a) * g(b)) = f(g(a)) * f(g(b))
            compose(f, g)(a * b) = compose(f, g)(a) * compose(f, g)(b)
        }
        is_monoid_hom(compose(f, g))
    }
}

/// The identity homomorphism on a monoid.
let identity_monoid_hom[M: Monoid]: MonoidHom[M, M] satisfy {
    MonoidHom.new(identity_fn[M]) = Option.some(identity_monoid_hom)
}

/// The underlying function of the identity monoid homomorphism is the identity function.
theorem identity_monoid_hom_hom[M: Monoid] {
    identity_monoid_hom[M].hom = identity_fn[M]
}

/// The composition of two monoid homomorphisms.
let compose_monoid_hom[M: Monoid, N: Monoid, P: Monoid](f: MonoidHom[N, P], g: MonoidHom[M, N]) -> result: MonoidHom[M, P] satisfy {
    MonoidHom.new(compose(f.hom, g.hom)) = Option.some(result)
} by {
    compose_is_monoid_hom(f.hom, g.hom)
}

/// The underlying function of a composition of monoid homomorphisms is the composition of underlying functions.
theorem compose_monoid_hom_hom[M: Monoid, N: Monoid, P: Monoid](
    f: MonoidHom[N, P],
    g: MonoidHom[M, N]
) {
    compose_monoid_hom(f, g).hom = compose(f.hom, g.hom)
}

/// Composition of monoid homomorphisms is associative.
theorem compose_monoid_hom_assoc[A: Monoid, B: Monoid, C: Monoid, D: Monoid](
    f: MonoidHom[C, D],
    g: MonoidHom[B, C],
    h: MonoidHom[A, B]
) {
    compose_monoid_hom(compose_monoid_hom(f, g), h) = compose_monoid_hom(f, compose_monoid_hom(g, h))
} by {
    let lhs = compose_monoid_hom(compose_monoid_hom(f, g), h)
    let rhs = compose_monoid_hom(f, compose_monoid_hom(g, h))
    lhs.hom = compose(compose_monoid_hom(f, g).hom, h.hom)
    compose_monoid_hom(f, g).hom = compose(f.hom, g.hom)
    lhs.hom = compose(compose(f.hom, g.hom), h.hom)
    rhs.hom = compose(f.hom, compose_monoid_hom(g, h).hom)
    compose_monoid_hom(g, h).hom = compose(g.hom, h.hom)
    rhs.hom = compose(f.hom, compose(g.hom, h.hom))
    compose_assoc(f.hom, g.hom, h.hom)
    lhs.hom = rhs.hom
}

/// Composing the identity monoid homomorphism on the left leaves a homomorphism unchanged.
theorem compose_monoid_hom_identity_left[M: Monoid, N: Monoid](f: MonoidHom[M, N]) {
    compose_monoid_hom(identity_monoid_hom[N], f) = f
} by {
    let lhs = compose_monoid_hom(identity_monoid_hom[N], f)
    compose_identity_left(f.hom)
}

/// Composing the identity monoid homomorphism on the right leaves a homomorphism unchanged.
theorem compose_monoid_hom_identity_right[M: Monoid, N: Monoid](f: MonoidHom[M, N]) {
    compose_monoid_hom(f, identity_monoid_hom[M]) = f
} by {
    let lhs = compose_monoid_hom(f, identity_monoid_hom[M])
    compose_identity_right(f.hom)
}
