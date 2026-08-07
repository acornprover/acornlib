from category.category import Category, is_category, category_id_endpoints_constraint,
    category_compose_src_constraint, category_compose_dst_constraint,
    category_assoc_constraint, category_id_left_constraint,
    category_id_right_constraint, category_new_src, category_new_dst,
    category_new_identity, category_new_compose
from algebra.monoid.monoid import Monoid

/// The unique object of the category associated to a monoid.
inductive OneObject {
    /// The only object.
    point
}

/// Every value of `OneObject` is the unique point.
theorem one_object_unique(x: OneObject) {
    x = OneObject.point
} by {
    match x {
        OneObject.point {
            x = OneObject.point
        }
    }
}

/// Any two values of `OneObject` are equal.
theorem one_object_eq(x: OneObject, y: OneObject) {
    x = y
} by {
    one_object_unique(x)
    one_object_unique(y)
}

/// The source map of the one-object category associated to a monoid.
define monoid_category_src[M: Monoid](m: M) -> OneObject {
    OneObject.point
}

/// The target map of the one-object category associated to a monoid.
define monoid_category_dst[M: Monoid](m: M) -> OneObject {
    OneObject.point
}

/// The identity morphism at the unique object is the monoid unit.
define monoid_category_identity[M: Monoid](x: OneObject) -> M {
    M.1
}

/// Composition is monoid multiplication, with `compose(f, g) = f * g`
/// matching the convention that `compose(f, g)` means `f` after `g`.
define monoid_category_compose[M: Monoid](f: M, g: M) -> M {
    f * g
}

/// The one-object data associated to a monoid satisfies the category axioms.
theorem monoid_category_is_category[M: Monoid] {
    is_category[OneObject, M](
        monoid_category_src[M],
        monoid_category_dst[M],
        monoid_category_identity[M],
        monoid_category_compose[M]
    )
} by {
    forall(x: OneObject) {
        one_object_unique(x)
        monoid_category_dst[M](monoid_category_identity[M](x)) = x
    }
    category_id_endpoints_constraint(
        monoid_category_src[M],
        monoid_category_dst[M],
        monoid_category_identity[M]
    )

    forall(f: M, g: M) {
        if monoid_category_src[M](f) = monoid_category_dst[M](g) {
            monoid_category_src[M](monoid_category_compose[M](f, g)) = monoid_category_src[M](g)
        }
    }
    category_compose_src_constraint(
        monoid_category_src[M],
        monoid_category_dst[M],
        monoid_category_compose[M]
    )

    forall(f: M, g: M) {
        if monoid_category_src[M](f) = monoid_category_dst[M](g) {
            monoid_category_dst[M](monoid_category_compose[M](f, g)) = monoid_category_dst[M](f)
        }
    }
    category_compose_dst_constraint(
        monoid_category_src[M],
        monoid_category_dst[M],
        monoid_category_compose[M]
    )

    forall(f: M, g: M, h: M) {
        if monoid_category_src[M](f) = monoid_category_dst[M](g)
            and monoid_category_src[M](g) = monoid_category_dst[M](h) {
            f * (g * h) = (f * g) * h
            monoid_category_compose[M](monoid_category_compose[M](f, g), h) =
                monoid_category_compose[M](f, monoid_category_compose[M](g, h))
        }
    }
    category_assoc_constraint(
        monoid_category_src[M],
        monoid_category_dst[M],
        monoid_category_compose[M]
    )

    forall(f: M) {
        monoid_category_compose[M](monoid_category_identity[M](monoid_category_dst[M](f)), f) = f
    }
    category_id_left_constraint(
        monoid_category_src[M],
        monoid_category_dst[M],
        monoid_category_identity[M],
        monoid_category_compose[M]
    )

    forall(f: M) {
        monoid_category_compose[M](f, monoid_category_identity[M](monoid_category_src[M](f))) = f
    }
    category_id_right_constraint(
        monoid_category_src[M],
        monoid_category_dst[M],
        monoid_category_identity[M],
        monoid_category_compose[M]
    )
}

/// The category with one object and morphisms given by a monoid.
let monoid_category[M: Monoid]: Category[OneObject, M] satisfy {
    Category[OneObject, M].new(
        monoid_category_src[M],
        monoid_category_dst[M],
        monoid_category_identity[M],
        monoid_category_compose[M]
    ) = Option.some(monoid_category)
}

/// The bundled monoid category has the expected source map.
theorem monoid_category_src_eq[M: Monoid] {
    monoid_category[M].src = monoid_category_src[M]
} by {
    let c = monoid_category[M]
    category_new_src(
        monoid_category_src[M],
        monoid_category_dst[M],
        monoid_category_identity[M],
        monoid_category_compose[M],
        c
    )
}

/// The bundled monoid category has the expected target map.
theorem monoid_category_dst_eq[M: Monoid] {
    monoid_category[M].dst = monoid_category_dst[M]
} by {
    let c = monoid_category[M]
    category_new_dst(
        monoid_category_src[M],
        monoid_category_dst[M],
        monoid_category_identity[M],
        monoid_category_compose[M],
        c
    )
}

/// The bundled monoid category has the expected identity map.
theorem monoid_category_identity_eq[M: Monoid] {
    monoid_category[M].identity = monoid_category_identity[M]
} by {
    let c = monoid_category[M]
    Category[OneObject, M].new(
        monoid_category_src[M],
        monoid_category_dst[M],
        monoid_category_identity[M],
        monoid_category_compose[M]
    ) = Option.some(c)
    category_new_identity(
        monoid_category_src[M],
        monoid_category_dst[M],
        monoid_category_identity[M],
        monoid_category_compose[M],
        c
    )
}

/// The bundled monoid category has the expected composition map.
theorem monoid_category_compose_eq[M: Monoid] {
    monoid_category[M].compose = monoid_category_compose[M]
} by {
    let c = monoid_category[M]
    Category[OneObject, M].new(
        monoid_category_src[M],
        monoid_category_dst[M],
        monoid_category_identity[M],
        monoid_category_compose[M]
    ) = Option.some(c)
    category_new_compose(
        monoid_category_src[M],
        monoid_category_dst[M],
        monoid_category_identity[M],
        monoid_category_compose[M],
        c
    )
}

/// Every morphism has the unique object as source.
theorem monoid_category_src_apply[M: Monoid](m: M) {
    monoid_category[M].src(m) = OneObject.point
} by {
    monoid_category_src_eq[M]
}

/// Every morphism has the unique object as target.
theorem monoid_category_dst_apply[M: Monoid](m: M) {
    monoid_category[M].dst(m) = OneObject.point
} by {
    monoid_category_dst_eq[M]
}

/// The identity at the unique object is the monoid unit.
theorem monoid_category_identity_apply[M: Monoid](x: OneObject) {
    monoid_category[M].identity(x) = M.1
} by {
    monoid_category_identity_eq[M]
}

/// Composition in the monoid category is monoid multiplication.
theorem monoid_category_compose_apply[M: Monoid](f: M, g: M) {
    monoid_category[M].compose(f, g) = f * g
} by {
    monoid_category_compose_eq[M]
}

/// Left identity in the bundled monoid category.
theorem monoid_category_id_left_apply[M: Monoid](f: M) {
    monoid_category[M].compose(monoid_category[M].identity(OneObject.point), f) = f
} by {
    monoid_category_identity_apply[M](OneObject.point)
    monoid_category_compose_apply[M](monoid_category[M].identity(OneObject.point), f)
}

/// Right identity in the bundled monoid category.
theorem monoid_category_id_right_apply[M: Monoid](f: M) {
    monoid_category[M].compose(f, monoid_category[M].identity(OneObject.point)) = f
} by {
    monoid_category_identity_apply[M](OneObject.point)
    monoid_category_compose_apply[M](f, monoid_category[M].identity(OneObject.point))
}

/// Associativity in the bundled monoid category.
theorem monoid_category_compose_assoc_apply[M: Monoid](f: M, g: M, h: M) {
    monoid_category[M].compose(monoid_category[M].compose(f, g), h) =
        monoid_category[M].compose(f, monoid_category[M].compose(g, h))
} by {
    monoid_category_compose_apply[M](f, g)
    monoid_category_compose_apply[M](g, h)
    monoid_category_compose_apply[M](monoid_category[M].compose(f, g), h)
    monoid_category_compose_apply[M](f, monoid_category[M].compose(g, h))
    f * (g * h) = (f * g) * h
}
