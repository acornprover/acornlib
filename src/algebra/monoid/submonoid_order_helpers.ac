from algebra.monoid.monoid import Monoid
from data.basic.set import Set, subset_contains
from algebra.monoid.submonoid import Submonoid, submonoid_subset, submonoid_subset_as_set_eq,
    submonoid_subset_refl, submonoid_subset_trans, submonoid_subset_antisymm,
    submonoid_intersection_subset_left_relation, submonoid_intersection_subset_right_relation,
    submonoid_subset_intersection_of_subset_left_right, submonoid_subset_intersection_iff,
    submonoid_closure, submonoid_closure_mono, submonoid_closure_le_iff_set_subset,
    set_subset_submonoid_as_set_eq, submonoid_sup, submonoid_subset_sup_left,
    submonoid_subset_sup_right, submonoid_sup_subset_of_subset_left_right

/// Submonoid inclusion is exactly elementwise implication of membership predicates.
theorem submonoid_subset_contains_eq[M: Monoid](a: Submonoid[M], b: Submonoid[M]) {
    submonoid_subset(a, b) = forall(x: M) {
        a.contains(x) implies b.contains(x)
    }
} by {
    submonoid_subset(a, b) = forall(x: M) {
        a.contains(x) implies b.contains(x)
    }
}

/// A submonoid inclusion transports membership from the smaller submonoid to the larger one.
theorem submonoid_subset_contains[M: Monoid](a: Submonoid[M], b: Submonoid[M], x: M) {
    submonoid_subset(a, b) and a.contains(x) implies b.contains(x)
} by {
    if submonoid_subset(a, b) and a.contains(x) {
        submonoid_subset_contains_eq(a, b)
        a.contains(x) implies b.contains(x)
        b.contains(x)
    }
}

/// Elementwise membership implication gives submonoid inclusion.
theorem submonoid_subset_of_contains[M: Monoid](a: Submonoid[M], b: Submonoid[M]) {
    (forall(x: M) { a.contains(x) implies b.contains(x) }) implies submonoid_subset(a, b)
} by {
    if forall(x: M) { a.contains(x) implies b.contains(x) } {
        submonoid_subset_contains_eq(a, b)
        submonoid_subset(a, b)
    }
}

/// Submonoid inclusion is reflexive.
theorem submonoid_order_subset_refl[M: Monoid](a: Submonoid[M]) {
    submonoid_subset(a, a)
} by {
    submonoid_subset_refl(a)
}

/// Submonoid inclusion is transitive.
theorem submonoid_order_subset_trans[M: Monoid](
    a: Submonoid[M],
    b: Submonoid[M],
    c: Submonoid[M]
) {
    submonoid_subset(a, b) and submonoid_subset(b, c) implies submonoid_subset(a, c)
} by {
    if submonoid_subset(a, b) and submonoid_subset(b, c) {
        submonoid_subset_trans(a, b, c)
    }
}

/// Mutual submonoid inclusion forces equality.
theorem submonoid_order_subset_antisymm[M: Monoid](a: Submonoid[M], b: Submonoid[M]) {
    submonoid_subset(a, b) and submonoid_subset(b, a) implies a = b
} by {
    if submonoid_subset(a, b) and submonoid_subset(b, a) {
        submonoid_subset_antisymm(a, b)
    }
}

/// Submonoid inclusion is exactly inclusion of the underlying sets.
theorem submonoid_subset_as_set_iff[M: Monoid](a: Submonoid[M], b: Submonoid[M]) {
    submonoid_subset(a, b) = a.as_set.subset(b.as_set)
} by {
    submonoid_subset_as_set_eq(a, b)
}

/// Underlying-set inclusion transports submonoid membership.
theorem submonoid_as_set_subset_contains[M: Monoid](a: Submonoid[M], b: Submonoid[M], x: M) {
    a.as_set.subset(b.as_set) and a.contains(x) implies b.contains(x)
} by {
    if a.as_set.subset(b.as_set) and a.contains(x) {
        a.as_set.contains(x)
        subset_contains(a.as_set, b.as_set, x)
        b.as_set.contains(x)
        submonoid_subset_as_set_eq(a, b)
        submonoid_subset(a, b)
        submonoid_subset_contains(a, b, x)
    }
}

/// The intersection is the greatest lower bound for submonoid inclusion.
theorem submonoid_intersection_greatest_lower_bound[M: Monoid](
    c: Submonoid[M],
    a: Submonoid[M],
    b: Submonoid[M]
) {
    submonoid_subset(c, a.intersection(b)) = (submonoid_subset(c, a) and submonoid_subset(c, b))
} by {
    submonoid_subset_intersection_iff(c, a, b)
}

/// A submonoid contained in an intersection is contained in the left factor.
theorem submonoid_subset_left_of_subset_intersection[M: Monoid](
    c: Submonoid[M],
    a: Submonoid[M],
    b: Submonoid[M]
) {
    submonoid_subset(c, a.intersection(b)) implies submonoid_subset(c, a)
} by {
    if submonoid_subset(c, a.intersection(b)) {
        submonoid_intersection_subset_left_relation(a, b)
        submonoid_subset_trans(c, a.intersection(b), a)
        submonoid_subset(c, a)
    }
}

/// A submonoid contained in an intersection is contained in the right factor.
theorem submonoid_subset_right_of_subset_intersection[M: Monoid](
    c: Submonoid[M],
    a: Submonoid[M],
    b: Submonoid[M]
) {
    submonoid_subset(c, a.intersection(b)) implies submonoid_subset(c, b)
} by {
    if submonoid_subset(c, a.intersection(b)) {
        submonoid_intersection_subset_right_relation(a, b)
        submonoid_subset_trans(c, a.intersection(b), b)
        submonoid_subset(c, b)
    }
}

/// A common submonoid of two submonoids is contained in their intersection.
theorem submonoid_subset_intersection_of_subset_both[M: Monoid](
    c: Submonoid[M],
    a: Submonoid[M],
    b: Submonoid[M]
) {
    submonoid_subset(c, a) and submonoid_subset(c, b) implies submonoid_subset(c, a.intersection(b))
} by {
    if submonoid_subset(c, a) and submonoid_subset(c, b) {
        submonoid_subset_intersection_of_subset_left_right(c, a, b)
    }
}

/// Submonoid closure is monotone with respect to inclusion of generating sets.
theorem submonoid_closure_monotone[M: Monoid](a: Set[M], b: Set[M]) {
    a.subset(b) implies submonoid_subset(submonoid_closure(a), submonoid_closure(b))
} by {
    if a.subset(b) {
        submonoid_closure_mono(a, b)
    }
}

/// The underlying set of submonoid closure is monotone with respect to inclusion of generating sets.
theorem submonoid_closure_as_set_monotone[M: Monoid](a: Set[M], b: Set[M]) {
    a.subset(b) implies submonoid_closure(a).as_set.subset(submonoid_closure(b).as_set)
} by {
    if a.subset(b) {
        submonoid_closure_monotone(a, b)
        submonoid_subset_as_set_eq(submonoid_closure(a), submonoid_closure(b))
        submonoid_closure(a).as_set.subset(submonoid_closure(b).as_set)
    }
}

/// The closure of a set is contained in a submonoid exactly when the set is contained in its underlying set.
theorem submonoid_closure_subset_iff_as_set_subset[M: Monoid](a: Set[M], s: Submonoid[M]) {
    submonoid_subset(submonoid_closure(a), s) = a.subset(s.as_set)
} by {
    submonoid_closure_le_iff_set_subset(a, s)
    set_subset_submonoid_as_set_eq(a, s)
}

/// Mutual inclusion of submonoids is equivalent to equality.
theorem submonoid_subset_antisymm_iff_eq[M: Monoid](a: Submonoid[M], b: Submonoid[M]) {
    (submonoid_subset(a, b) and submonoid_subset(b, a)) = (a = b)
} by {
    if submonoid_subset(a, b) and submonoid_subset(b, a) {
        submonoid_subset_antisymm(a, b)
        a = b
    }
    if a = b {
        submonoid_subset_refl(a)
        submonoid_subset(a, b)
        submonoid_subset(b, a)
    }
    (submonoid_subset(a, b) and submonoid_subset(b, a)) = (a = b)
}

/// Intersections of submonoids are monotone in the left argument.
theorem submonoid_intersection_mono_left[M: Monoid](
    a: Submonoid[M],
    b: Submonoid[M],
    c: Submonoid[M]
) {
    submonoid_subset(a, b) implies submonoid_subset(a.intersection(c), b.intersection(c))
} by {
    if submonoid_subset(a, b) {
        submonoid_intersection_subset_left_relation(a, c)
        submonoid_subset_trans(a.intersection(c), a, b)
        submonoid_subset(a.intersection(c), b)
        submonoid_intersection_subset_right_relation(a, c)
        submonoid_subset(a.intersection(c), c)
        submonoid_subset_intersection_of_subset_left_right(a.intersection(c), b, c)
        submonoid_subset(a.intersection(c), b.intersection(c))
    }
}

/// Intersections of submonoids are monotone in the right argument.
theorem submonoid_intersection_mono_right[M: Monoid](
    a: Submonoid[M],
    b: Submonoid[M],
    c: Submonoid[M]
) {
    submonoid_subset(a, b) implies submonoid_subset(c.intersection(a), c.intersection(b))
} by {
    if submonoid_subset(a, b) {
        submonoid_intersection_subset_left_relation(c, a)
        submonoid_subset(c.intersection(a), c)
        submonoid_intersection_subset_right_relation(c, a)
        submonoid_subset_trans(c.intersection(a), a, b)
        submonoid_subset(c.intersection(a), b)
        submonoid_subset_intersection_of_subset_left_right(c.intersection(a), c, b)
        submonoid_subset(c.intersection(a), c.intersection(b))
    }
}

/// Intersections of submonoids are monotone in both arguments.
theorem submonoid_intersection_mono[M: Monoid](
    a: Submonoid[M],
    b: Submonoid[M],
    c: Submonoid[M],
    d: Submonoid[M]
) {
    submonoid_subset(a, b) and submonoid_subset(c, d) implies
        submonoid_subset(a.intersection(c), b.intersection(d))
} by {
    if submonoid_subset(a, b) and submonoid_subset(c, d) {
        submonoid_intersection_mono_left(a, b, c)
        submonoid_subset(a.intersection(c), b.intersection(c))
        submonoid_intersection_mono_right(c, d, b)
        submonoid_subset(b.intersection(c), b.intersection(d))
        submonoid_subset_trans(a.intersection(c), b.intersection(c), b.intersection(d))
        submonoid_subset(a.intersection(c), b.intersection(d))
    }
}

/// Joins of submonoids are monotone in the left argument.
theorem submonoid_sup_mono_left[M: Monoid](
    a: Submonoid[M],
    b: Submonoid[M],
    c: Submonoid[M]
) {
    submonoid_subset(a, b) implies submonoid_subset(submonoid_sup(a, c), submonoid_sup(b, c))
} by {
    if submonoid_subset(a, b) {
        submonoid_subset_sup_left(b, c)
        submonoid_subset_trans(a, b, submonoid_sup(b, c))
        submonoid_subset(a, submonoid_sup(b, c))
        submonoid_subset_sup_right(b, c)
        submonoid_subset(c, submonoid_sup(b, c))
        submonoid_sup_subset_of_subset_left_right(a, c, submonoid_sup(b, c))
        submonoid_subset(submonoid_sup(a, c), submonoid_sup(b, c))
    }
}

/// Joins of submonoids are monotone in the right argument.
theorem submonoid_sup_mono_right[M: Monoid](
    a: Submonoid[M],
    b: Submonoid[M],
    c: Submonoid[M]
) {
    submonoid_subset(a, b) implies submonoid_subset(submonoid_sup(c, a), submonoid_sup(c, b))
} by {
    if submonoid_subset(a, b) {
        submonoid_subset_sup_left(c, b)
        submonoid_subset(c, submonoid_sup(c, b))
        submonoid_subset_sup_right(c, b)
        submonoid_subset_trans(a, b, submonoid_sup(c, b))
        submonoid_subset(a, submonoid_sup(c, b))
        submonoid_sup_subset_of_subset_left_right(c, a, submonoid_sup(c, b))
        submonoid_subset(submonoid_sup(c, a), submonoid_sup(c, b))
    }
}

/// Joins of submonoids are monotone in both arguments.
theorem submonoid_sup_mono[M: Monoid](
    a: Submonoid[M],
    b: Submonoid[M],
    c: Submonoid[M],
    d: Submonoid[M]
) {
    submonoid_subset(a, b) and submonoid_subset(c, d) implies
        submonoid_subset(submonoid_sup(a, c), submonoid_sup(b, d))
} by {
    if submonoid_subset(a, b) and submonoid_subset(c, d) {
        submonoid_subset_sup_left(b, d)
        submonoid_subset_trans(a, b, submonoid_sup(b, d))
        submonoid_subset(a, submonoid_sup(b, d))
        submonoid_subset_sup_right(b, d)
        submonoid_subset_trans(c, d, submonoid_sup(b, d))
        submonoid_subset(c, submonoid_sup(b, d))
        submonoid_sup_subset_of_subset_left_right(a, c, submonoid_sup(b, d))
        submonoid_subset(submonoid_sup(a, c), submonoid_sup(b, d))
    }
}

/// Mutual containment in generated submonoids identifies two submonoid closures.
theorem submonoid_closure_eq_of_mutual_as_set_subset[M: Monoid](a: Set[M], b: Set[M]) {
    a.subset(submonoid_closure(b).as_set) and b.subset(submonoid_closure(a).as_set) implies
        submonoid_closure(a) = submonoid_closure(b)
} by {
    if a.subset(submonoid_closure(b).as_set) and b.subset(submonoid_closure(a).as_set) {
        submonoid_closure_subset_iff_as_set_subset(a, submonoid_closure(b))
        submonoid_subset(submonoid_closure(a), submonoid_closure(b))
        submonoid_closure_subset_iff_as_set_subset(b, submonoid_closure(a))
        submonoid_subset(submonoid_closure(b), submonoid_closure(a))
        submonoid_subset_antisymm(submonoid_closure(a), submonoid_closure(b))
        submonoid_closure(a) = submonoid_closure(b)
    }
}
