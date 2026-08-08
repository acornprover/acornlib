from algebra.add_comm_group import AddCommGroup, AddCommGroupHom, compose_add_comm_group_hom, compose_add_comm_group_hom_hom
from data.basic.functions import compose
from data.basic.set import Set, set_image, set_preimage, maps_into_set_image, set_image_contains_witness,
    set_image_subset_iff_subset_preimage, set_image_compose, set_preimage_compose,
    set_image_preimage_closure, set_image_preimage_closure_extensive,
    set_image_preimage_closure_monotone, set_image_preimage_closure_idempotent,
    set_preimage_image_kernel, set_preimage_image_kernel_monotone,
    set_preimage_image_kernel_reductive, set_preimage_image_kernel_idempotent,
    is_subset_monotone_map, is_set_extensive_map, is_set_reductive_map,
    is_set_idempotent_map, is_set_closure_operator, is_set_kernel_operator

/// The image of a set under an additive commutative group homomorphism.
define add_comm_group_hom_image[A: AddCommGroup, B: AddCommGroup](f: AddCommGroupHom[A, B], s: Set[A]) -> Set[B] {
    set_image(s, f.hom)
}

/// The preimage of a set under an additive commutative group homomorphism.
define add_comm_group_hom_preimage[A: AddCommGroup, B: AddCommGroup](f: AddCommGroupHom[A, B], s: Set[B]) -> Set[A] {
    set_preimage(f.hom, s)
}

/// The source closure induced by image followed by preimage along an additive commutative group homomorphism.
define add_comm_group_hom_image_preimage_closure[A: AddCommGroup, B: AddCommGroup](
    f: AddCommGroupHom[A, B],
    s: Set[A]
) -> Set[A] {
    set_image_preimage_closure(f.hom, s)
}

/// The target kernel induced by preimage followed by image along an additive commutative group homomorphism.
define add_comm_group_hom_preimage_image_kernel[A: AddCommGroup, B: AddCommGroup](
    f: AddCommGroupHom[A, B],
    s: Set[B]
) -> Set[B] {
    set_preimage_image_kernel(f.hom, s)
}

/// Membership in the image of an additive commutative group homomorphism has a source witness.
theorem add_comm_group_hom_image_contains_witness[A: AddCommGroup, B: AddCommGroup](
    f: AddCommGroupHom[A, B],
    s: Set[A],
    y: B
) {
    add_comm_group_hom_image(f, s).contains(y) implies exists(x: A) {
        s.contains(x) and y = f.hom(x)
    }
} by {
    set_image_contains_witness(s, f.hom, y)
}

/// A member of a set maps into its image under an additive commutative group homomorphism.
theorem add_comm_group_hom_maps_into_image[A: AddCommGroup, B: AddCommGroup](
    f: AddCommGroupHom[A, B],
    s: Set[A],
    x: A
) {
    s.contains(x) implies add_comm_group_hom_image(f, s).contains(f.hom(x))
} by {
    if s.contains(x) {
        maps_into_set_image(s, f.hom, x)
        add_comm_group_hom_image(f, s) = set_image(s, f.hom)
    }
}

/// Image containment is equivalent to source containment in the preimage.
theorem add_comm_group_hom_image_subset_iff_subset_preimage[A: AddCommGroup, B: AddCommGroup](
    f: AddCommGroupHom[A, B],
    s: Set[A],
    t: Set[B]
) {
    add_comm_group_hom_image(f, s).subset(t) = s.subset(add_comm_group_hom_preimage(f, t))
} by {
    set_image_subset_iff_subset_preimage(s, t, f.hom)
}

/// Image under a composite additive commutative group homomorphism is iterated image.
theorem add_comm_group_hom_image_compose[A: AddCommGroup, B: AddCommGroup, C: AddCommGroup](
    f: AddCommGroupHom[B, C],
    g: AddCommGroupHom[A, B],
    s: Set[A]
) {
    add_comm_group_hom_image(compose_add_comm_group_hom(f, g), s) =
        add_comm_group_hom_image(f, add_comm_group_hom_image(g, s))
} by {
    compose_add_comm_group_hom_hom(f, g)
    set_image_compose(s, g.hom, f.hom)
}

/// Preimage under a composite additive commutative group homomorphism is iterated preimage.
theorem add_comm_group_hom_preimage_compose[A: AddCommGroup, B: AddCommGroup, C: AddCommGroup](
    f: AddCommGroupHom[B, C],
    g: AddCommGroupHom[A, B],
    s: Set[C]
) {
    add_comm_group_hom_preimage(compose_add_comm_group_hom(f, g), s) =
        add_comm_group_hom_preimage(g, add_comm_group_hom_preimage(f, s))
} by {
    compose_add_comm_group_hom_hom(f, g)
    set_preimage_compose(f.hom, g.hom, s)
}

/// The source closure induced by an additive commutative group homomorphism contains the original set.
theorem add_comm_group_hom_image_preimage_closure_extensive[A: AddCommGroup, B: AddCommGroup](
    f: AddCommGroupHom[A, B],
    s: Set[A]
) {
    s.subset(add_comm_group_hom_image_preimage_closure(f, s))
} by {
    set_image_preimage_closure_extensive(f.hom, s)
}

/// The source closure induced by an additive commutative group homomorphism preserves inclusion.
theorem add_comm_group_hom_image_preimage_closure_monotone[A: AddCommGroup, B: AddCommGroup](
    f: AddCommGroupHom[A, B]
) {
    is_subset_monotone_map(add_comm_group_hom_image_preimage_closure(f))
} by {
    forall(s: Set[A], t: Set[A]) {
        if s.subset(t) {
            set_image_preimage_closure_monotone(f.hom, s, t)
            add_comm_group_hom_image_preimage_closure(f, s).subset(
                add_comm_group_hom_image_preimage_closure(f, t)
            )
        }
    }
}

/// The source closure induced by an additive commutative group homomorphism is idempotent.
theorem add_comm_group_hom_image_preimage_closure_idempotent[A: AddCommGroup, B: AddCommGroup](
    f: AddCommGroupHom[A, B],
    s: Set[A]
) {
    add_comm_group_hom_image_preimage_closure(f, add_comm_group_hom_image_preimage_closure(f, s)) =
        add_comm_group_hom_image_preimage_closure(f, s)
} by {
    set_image_preimage_closure_idempotent(f.hom, s)
}

/// The source closure induced by an additive commutative group homomorphism is a set closure operator.
theorem add_comm_group_hom_image_preimage_closure_operator[A: AddCommGroup, B: AddCommGroup](
    f: AddCommGroupHom[A, B]
) {
    is_set_closure_operator(add_comm_group_hom_image_preimage_closure(f))
} by {
    add_comm_group_hom_image_preimage_closure_monotone(f)
    forall(s: Set[A]) {
        add_comm_group_hom_image_preimage_closure_extensive(f, s)
        s.subset(add_comm_group_hom_image_preimage_closure(f, s))
    }
    forall(s: Set[A]) {
        add_comm_group_hom_image_preimage_closure_idempotent(f, s)
        add_comm_group_hom_image_preimage_closure(f, add_comm_group_hom_image_preimage_closure(f, s)) =
            add_comm_group_hom_image_preimage_closure(f, s)
    }
}

/// The target kernel induced by an additive commutative group homomorphism lies in the original set.
theorem add_comm_group_hom_preimage_image_kernel_reductive[A: AddCommGroup, B: AddCommGroup](
    f: AddCommGroupHom[A, B],
    s: Set[B]
) {
    add_comm_group_hom_preimage_image_kernel(f, s).subset(s)
} by {
    set_preimage_image_kernel_reductive(f.hom, s)
}

/// The target kernel induced by an additive commutative group homomorphism preserves inclusion.
theorem add_comm_group_hom_preimage_image_kernel_monotone[A: AddCommGroup, B: AddCommGroup](
    f: AddCommGroupHom[A, B]
) {
    is_subset_monotone_map(add_comm_group_hom_preimage_image_kernel(f))
} by {
    forall(s: Set[B], t: Set[B]) {
        if s.subset(t) {
            set_preimage_image_kernel_monotone(f.hom, s, t)
            add_comm_group_hom_preimage_image_kernel(f, s).subset(
                add_comm_group_hom_preimage_image_kernel(f, t)
            )
        }
    }
}

/// The target kernel induced by an additive commutative group homomorphism is idempotent.
theorem add_comm_group_hom_preimage_image_kernel_idempotent[A: AddCommGroup, B: AddCommGroup](
    f: AddCommGroupHom[A, B],
    s: Set[B]
) {
    add_comm_group_hom_preimage_image_kernel(f, add_comm_group_hom_preimage_image_kernel(f, s)) =
        add_comm_group_hom_preimage_image_kernel(f, s)
} by {
    set_preimage_image_kernel_idempotent(f.hom, s)
}

/// The target kernel induced by an additive commutative group homomorphism is a set kernel operator.
theorem add_comm_group_hom_preimage_image_kernel_operator[A: AddCommGroup, B: AddCommGroup](
    f: AddCommGroupHom[A, B]
) {
    is_set_kernel_operator(add_comm_group_hom_preimage_image_kernel(f))
} by {
    add_comm_group_hom_preimage_image_kernel_monotone(f)
    forall(s: Set[B]) {
        add_comm_group_hom_preimage_image_kernel_reductive(f, s)
        add_comm_group_hom_preimage_image_kernel(f, s).subset(s)
    }
    is_set_reductive_map(add_comm_group_hom_preimage_image_kernel(f))
    forall(s: Set[B]) {
        add_comm_group_hom_preimage_image_kernel_idempotent(f, s)
        add_comm_group_hom_preimage_image_kernel(f, add_comm_group_hom_preimage_image_kernel(f, s)) =
            add_comm_group_hom_preimage_image_kernel(f, s)
    }
    is_set_idempotent_map(add_comm_group_hom_preimage_image_kernel(f))
}
