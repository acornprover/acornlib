from algebra.add_semigroup import AddSemigroup

/// Extending the additive semigroup with commutativity.
typeclass A: AddCommSemigroup extends AddSemigroup {
    /// The addition operation must be commutative: `a + b = b + a`.
    add_commutative(a: A, b: A) {
        a + b = b + a
    }
}