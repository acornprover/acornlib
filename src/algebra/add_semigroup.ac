from algebra.add import Add

/// An additive semigroup is associative, and that's about it.
typeclass A: AddSemigroup extends Add {
    /// The addition operation must be associative: `(a + b) + c = a + (b + c)`.
    add_associative(a: A, b: A, c: A) {
        a + (b + c) = (a + b) + c
    }
}

/// Pointwise addition of two functions.
define add_fn[T, A: AddSemigroup](f: T -> A, g: T -> A, t: T) -> A {
    f(t) + g(t)
}
