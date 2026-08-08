from algebra.comm_semigroup import CommSemigroup
from algebra.monoid.monoid import Monoid

/// CommMonoid represents a commutative, multiplicative monoid.
typeclass M: CommMonoid extends CommSemigroup, Monoid {
    // All the properties are provided by the base typeclasses.
}
