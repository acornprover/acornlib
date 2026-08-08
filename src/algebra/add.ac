/// The "Add" typeclass just represents anything that has a + operator.
typeclass A: Add {
    add: (A, A) -> A
}