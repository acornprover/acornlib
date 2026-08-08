from algebra.field.field import Field

/// True if a subset contains the additive identity.
define subfield_zero_constraint[F: Field](contains: F -> Bool) -> Bool {
    contains(F.0)
}

/// True if a subset contains the multiplicative identity.
define subfield_one_constraint[F: Field](contains: F -> Bool) -> Bool {
    contains(F.1)
}

/// True if a subset is closed under addition.
define subfield_add_constraint[F: Field](contains: F -> Bool) -> Bool {
    forall(a: F, b: F) {
        contains(a) and contains(b) implies contains(a + b)
    }
}

/// True if a subset is closed under additive negation.
define subfield_neg_constraint[F: Field](contains: F -> Bool) -> Bool {
    forall(a: F) {
        contains(a) implies contains(-a)
    }
}

/// True if a subset is closed under multiplication.
define subfield_mul_constraint[F: Field](contains: F -> Bool) -> Bool {
    forall(a: F, b: F) {
        contains(a) and contains(b) implies contains(a * b)
    }
}

/// True if a subset is closed under multiplicative inverses of nonzero elements.
define subfield_inverse_constraint[F: Field](contains: F -> Bool) -> Bool {
    forall(a: F) {
        contains(a) and a != F.0 implies contains(a.inverse)
    }
}

/// True if a subset is closed under additive subfield operations.
define subfield_additive_constraint[F: Field](contains: F -> Bool) -> Bool {
    subfield_zero_constraint(contains) and subfield_add_constraint(contains) and
        subfield_neg_constraint(contains)
}

/// True if a subset is closed under multiplicative subfield operations.
define subfield_multiplicative_constraint[F: Field](contains: F -> Bool) -> Bool {
    subfield_one_constraint(contains) and subfield_mul_constraint(contains) and
        subfield_inverse_constraint(contains)
}

/// True if a subset satisfies all the requirements to be a subfield.
define subfield_constraint[F: Field](contains: F -> Bool) -> Bool {
    subfield_additive_constraint(contains) and subfield_multiplicative_constraint(contains)
}

/// A subfield of a field, represented as a subset containing zero and one and closed under field operations.
structure Subfield[F: Field] {
    /// True if the given element is a member of this subfield.
    contains: F -> Bool
} constraint {
    subfield_constraint(contains)
}

/// Subfield extensionality from equality of membership.
theorem subfield_ext[F: Field](a: Subfield[F], b: Subfield[F]) {
    (forall(x: F) { a.contains(x) = b.contains(x) }) implies a = b
} by {
    if forall(x: F) { a.contains(x) = b.contains(x) } {
        a.contains = b.contains
    }
}

/// Equal subfields have equal membership predicates.
theorem subfield_eq_contains[F: Field](a: Subfield[F], b: Subfield[F]) {
    a = b implies a.contains = b.contains
}

/// Every subfield's membership predicate satisfies the subfield constraint.
theorem subfield_satisfies_constraint[F: Field](s: Subfield[F]) {
    subfield_constraint(s.contains)
}

/// Every subfield's membership predicate satisfies the additive constraints.
theorem subfield_satisfies_additive[F: Field](s: Subfield[F]) {
    subfield_additive_constraint(s.contains)
} by {
    subfield_satisfies_constraint(s)
}

/// Every subfield's membership predicate satisfies the multiplicative constraints.
theorem subfield_satisfies_multiplicative[F: Field](s: Subfield[F]) {
    subfield_multiplicative_constraint(s.contains)
} by {
    subfield_satisfies_constraint(s)
}

/// Every subfield's membership predicate satisfies the zero constraint.
theorem subfield_satisfies_zero[F: Field](s: Subfield[F]) {
    subfield_zero_constraint(s.contains)
} by {
    subfield_satisfies_additive(s)
    subfield_additive_constraint(s.contains) = subfield_zero_constraint(s.contains) and subfield_add_constraint(s.contains) and subfield_neg_constraint(s.contains)
}

/// Every subfield's membership predicate satisfies the one constraint.
theorem subfield_satisfies_one[F: Field](s: Subfield[F]) {
    subfield_one_constraint(s.contains)
} by {
    subfield_satisfies_multiplicative(s)
    subfield_multiplicative_constraint(s.contains) = subfield_one_constraint(s.contains) and subfield_mul_constraint(s.contains) and subfield_inverse_constraint(s.contains)
}

/// Every subfield's membership predicate satisfies the add constraint.
theorem subfield_satisfies_add[F: Field](s: Subfield[F]) {
    subfield_add_constraint(s.contains)
} by {
    subfield_satisfies_additive(s)
    subfield_additive_constraint(s.contains) = subfield_zero_constraint(s.contains) and subfield_add_constraint(s.contains) and subfield_neg_constraint(s.contains)
}

/// Every subfield's membership predicate satisfies the neg constraint.
theorem subfield_satisfies_neg[F: Field](s: Subfield[F]) {
    subfield_neg_constraint(s.contains)
} by {
    subfield_satisfies_additive(s)
    subfield_additive_constraint(s.contains) = subfield_zero_constraint(s.contains) and subfield_add_constraint(s.contains) and subfield_neg_constraint(s.contains)
}

/// Every subfield's membership predicate satisfies the mul constraint.
theorem subfield_satisfies_mul[F: Field](s: Subfield[F]) {
    subfield_mul_constraint(s.contains)
} by {
    subfield_satisfies_multiplicative(s)
    subfield_multiplicative_constraint(s.contains) = subfield_one_constraint(s.contains) and subfield_mul_constraint(s.contains) and subfield_inverse_constraint(s.contains)
}

/// Every subfield's membership predicate satisfies the inverse constraint.
theorem subfield_satisfies_inverse[F: Field](s: Subfield[F]) {
    subfield_inverse_constraint(s.contains)
} by {
    subfield_satisfies_multiplicative(s)
    subfield_multiplicative_constraint(s.contains) = subfield_one_constraint(s.contains) and subfield_mul_constraint(s.contains) and subfield_inverse_constraint(s.contains)
}

/// Membership of zero in any subfield.
theorem subfield_contains_zero[F: Field](s: Subfield[F]) {
    s.contains(F.0)
} by {
    subfield_satisfies_zero(s)
    subfield_zero_constraint(s.contains) = s.contains(F.0)
}

/// Membership of one in any subfield.
theorem subfield_contains_one[F: Field](s: Subfield[F]) {
    s.contains(F.1)
} by {
    subfield_satisfies_one(s)
    subfield_one_constraint(s.contains) = s.contains(F.1)
}

/// Closure of any subfield under addition.
theorem subfield_contains_add[F: Field](s: Subfield[F], a: F, b: F) {
    s.contains(a) and s.contains(b) implies s.contains(a + b)
} by {
    if s.contains(a) and s.contains(b) {
        subfield_satisfies_add(s)
        subfield_add_constraint(s.contains) = forall(x: F, y: F) {
            s.contains(x) and s.contains(y) implies s.contains(x + y)
        }
        s.contains(a + b)
    }
}

/// Closure of any subfield under additive negation.
theorem subfield_contains_neg[F: Field](s: Subfield[F], a: F) {
    s.contains(a) implies s.contains(-a)
} by {
    if s.contains(a) {
        subfield_satisfies_neg(s)
        subfield_neg_constraint(s.contains) = forall(x: F) {
            s.contains(x) implies s.contains(-x)
        }
        s.contains(-a)
    }
}

/// Closure of any subfield under subtraction.
theorem subfield_contains_sub[F: Field](s: Subfield[F], a: F, b: F) {
    s.contains(a) and s.contains(b) implies s.contains(a - b)
} by {
    if s.contains(a) and s.contains(b) {
        subfield_contains_neg(s, b)
        s.contains(-b)
        subfield_contains_add(s, a, -b)
        s.contains(a + -b)
        a - b = a + -b
    }
}

/// Closure of any subfield under multiplication.
theorem subfield_contains_mul[F: Field](s: Subfield[F], a: F, b: F) {
    s.contains(a) and s.contains(b) implies s.contains(a * b)
} by {
    if s.contains(a) and s.contains(b) {
        subfield_satisfies_mul(s)
        subfield_mul_constraint(s.contains) = forall(x: F, y: F) {
            s.contains(x) and s.contains(y) implies s.contains(x * y)
        }
        s.contains(a * b)
    }
}

/// True if every element of a is also an element of b.
define subfield_le[F: Field](a: Subfield[F], b: Subfield[F]) -> Bool {
    forall(x: F) { a.contains(x) implies b.contains(x) }
}

/// Introduction rule for subfield inclusion from pointwise membership.
theorem subfield_le_intro[F: Field](a: Subfield[F], b: Subfield[F]) {
    (forall(x: F) { a.contains(x) implies b.contains(x) }) implies subfield_le(a, b)
}

/// Subfield inclusion is reflexive.
theorem subfield_le_refl[F: Field](a: Subfield[F]) {
    subfield_le(a, a)
} by {
    forall(x: F) {
        if a.contains(x) {
            a.contains(x)
        }
    }
}

/// Subfield inclusion is transitive.
theorem subfield_le_trans[F: Field](a: Subfield[F], b: Subfield[F], c: Subfield[F]) {
    subfield_le(a, b) and subfield_le(b, c) implies subfield_le(a, c)
} by {
    if subfield_le(a, b) and subfield_le(b, c) {
        forall(x: F) {
            if a.contains(x) {
                b.contains(x)
                c.contains(x)
            }
        }
    }
}

/// Subfield inclusion is antisymmetric.
theorem subfield_le_antisymm[F: Field](a: Subfield[F], b: Subfield[F]) {
    subfield_le(a, b) and subfield_le(b, a) implies a = b
} by {
    if subfield_le(a, b) and subfield_le(b, a) {
        subfield_le(a, b) = forall(x: F) { a.contains(x) implies b.contains(x) }
        subfield_le(b, a) = forall(x: F) { b.contains(x) implies a.contains(x) }
        forall(x: F) {
            a.contains(x) implies b.contains(x)
            b.contains(x) implies a.contains(x)
            a.contains(x) = b.contains(x)
        }
        let p: Bool = forall(x: F) { a.contains(x) = b.contains(x) }
        p
        subfield_ext(a, b)
        a = b
    }
}

/// Closure of any subfield under multiplicative inverses of nonzero elements.
theorem subfield_contains_inverse[F: Field](s: Subfield[F], a: F) {
    s.contains(a) and a != F.0 implies s.contains(a.inverse)
} by {
    if s.contains(a) and a != F.0 {
        subfield_satisfies_inverse(s)
        subfield_inverse_constraint(s.contains) = forall(x: F) {
            s.contains(x) and x != F.0 implies s.contains(x.inverse)
        }
        s.contains(a.inverse)
    }
}

/// Closure of any subfield under multiplication with the inverse of a nonzero element.
theorem subfield_contains_mul_inverse[F: Field](s: Subfield[F], a: F, b: F) {
    s.contains(a) and s.contains(b) and b != F.0 implies s.contains(a * b.inverse)
} by {
    if s.contains(a) and s.contains(b) and b != F.0 {
        subfield_contains_inverse(s, b)
        s.contains(b.inverse)
        subfield_contains_mul(s, a, b.inverse)
    }
}

/// True if an element lies in both of two subsets.
define subfield_inter_contains[F: Field](a: F -> Bool, b: F -> Bool) -> F -> Bool {
    function(x: F) { a(x) and b(x) }
}

/// The intersection of two subfield membership predicates satisfies the subfield constraint.
theorem subfield_inter_is_subfield[F: Field](a: Subfield[F], b: Subfield[F]) {
    subfield_constraint(subfield_inter_contains(a.contains, b.contains))
} by {
    let contains: F -> Bool = subfield_inter_contains(a.contains, b.contains)
    forall(x: F) {
        contains(x) = (a.contains(x) and b.contains(x))
    }
    subfield_contains_zero(a)
    subfield_contains_zero(b)
    contains(F.0)
    subfield_zero_constraint(contains)
    subfield_contains_one(a)
    subfield_contains_one(b)
    contains(F.1)
    subfield_one_constraint(contains)
    forall(x: F, y: F) {
        if contains(x) and contains(y) {
            contains(x)
            contains(y)
            contains(x) = (a.contains(x) and b.contains(x))
            contains(y) = (a.contains(y) and b.contains(y))
            a.contains(x)
            a.contains(y)
            a.contains(x) and a.contains(y)
            subfield_contains_add(a, x, y)
            a.contains(x + y)
            b.contains(x)
            b.contains(y)
            b.contains(x) and b.contains(y)
            subfield_contains_add(b, x, y)
            b.contains(x + y)
            contains(x + y) = (a.contains(x + y) and b.contains(x + y))
            contains(x + y)
        }
    }
    subfield_add_constraint(contains)
    forall(x: F) {
        if contains(x) {
            contains(x) = (a.contains(x) and b.contains(x))
            a.contains(x)
            subfield_contains_neg(a, x)
            a.contains(-x)
            b.contains(x)
            subfield_contains_neg(b, x)
            b.contains(-x)
            contains(-x) = (a.contains(-x) and b.contains(-x))
            contains(-x)
        }
    }
    subfield_neg_constraint(contains)
    subfield_additive_constraint(contains)
    forall(x: F, y: F) {
        if contains(x) and contains(y) {
            contains(x)
            contains(y)
            contains(x) = (a.contains(x) and b.contains(x))
            contains(y) = (a.contains(y) and b.contains(y))
            a.contains(x)
            a.contains(y)
            a.contains(x) and a.contains(y)
            subfield_contains_mul(a, x, y)
            a.contains(x * y)
            b.contains(x)
            b.contains(y)
            b.contains(x) and b.contains(y)
            subfield_contains_mul(b, x, y)
            b.contains(x * y)
            a.contains(x * y) and b.contains(x * y)
            contains(x * y) = (a.contains(x * y) and b.contains(x * y))
            contains(x * y)
        }
    }
    subfield_mul_constraint(contains)
    forall(x: F) {
        if contains(x) and x != F.0 {
            contains(x)
            x != F.0
            contains(x) = (a.contains(x) and b.contains(x))
            a.contains(x)
            subfield_contains_inverse(a, x)
            a.contains(x.inverse)
            b.contains(x)
            subfield_contains_inverse(b, x)
            b.contains(x.inverse)
            a.contains(x.inverse) and b.contains(x.inverse)
            contains(x.inverse) = (a.contains(x.inverse) and b.contains(x.inverse))
            contains(x.inverse)
        }
    }
    subfield_inverse_constraint(contains)
    subfield_multiplicative_constraint(contains)
}

/// The intersection of two subfields, as a subfield.
let subfield_inter[F: Field](a: Subfield[F], b: Subfield[F]) -> result: Subfield[F] satisfy {
    Subfield.new(subfield_inter_contains(a.contains, b.contains)) = Option.some(result)
} by {
    subfield_inter_is_subfield(a, b)
}

/// Membership in an intersection of subfields is conjunction of memberships.
theorem subfield_inter_contains_eq[F: Field](a: Subfield[F], b: Subfield[F], x: F) {
    subfield_inter(a, b).contains(x) = (a.contains(x) and b.contains(x))
} by {
    subfield_inter(a, b).contains = subfield_inter_contains(a.contains, b.contains)
    subfield_inter_contains(a.contains, b.contains)(x) = (a.contains(x) and b.contains(x))
}

/// The intersection of two subfields is contained in the first.
theorem subfield_inter_le_left[F: Field](a: Subfield[F], b: Subfield[F]) {
    subfield_le(subfield_inter(a, b), a)
} by {
    forall(x: F) {
        if subfield_inter(a, b).contains(x) {
            subfield_inter_contains_eq(a, b, x)
            a.contains(x)
        }
    }
}

/// The intersection of two subfields is contained in the second.
theorem subfield_inter_le_right[F: Field](a: Subfield[F], b: Subfield[F]) {
    subfield_le(subfield_inter(a, b), b)
} by {
    forall(x: F) {
        if subfield_inter(a, b).contains(x) {
            subfield_inter_contains_eq(a, b, x)
            b.contains(x)
        }
    }
}

/// A subfield contained in two subfields is contained in their intersection.
theorem subfield_inter_greatest[F: Field](a: Subfield[F], b: Subfield[F], c: Subfield[F]) {
    subfield_le(c, a) and subfield_le(c, b) implies subfield_le(c, subfield_inter(a, b))
} by {
    if subfield_le(c, a) and subfield_le(c, b) {
        subfield_le(c, a) = forall(x: F) { c.contains(x) implies a.contains(x) }
        subfield_le(c, b) = forall(x: F) { c.contains(x) implies b.contains(x) }
        forall(x: F) {
            if c.contains(x) {
                a.contains(x)
                b.contains(x)
                subfield_inter_contains_eq(a, b, x)
                subfield_inter(a, b).contains(x)
            }
        }
    }
}

/// Subfield intersection is commutative.
theorem subfield_inter_comm[F: Field](a: Subfield[F], b: Subfield[F]) {
    subfield_inter(a, b) = subfield_inter(b, a)
} by {
    forall(x: F) {
        subfield_inter_contains_eq(a, b, x)
        subfield_inter_contains_eq(b, a, x)
        (a.contains(x) and b.contains(x)) = (b.contains(x) and a.contains(x))
        subfield_inter(a, b).contains(x) = subfield_inter(b, a).contains(x)
    }
    subfield_ext(subfield_inter(a, b), subfield_inter(b, a))
}

/// Subfield intersection is idempotent.
theorem subfield_inter_idem[F: Field](a: Subfield[F]) {
    subfield_inter(a, a) = a
} by {
    forall(x: F) {
        subfield_inter_contains_eq(a, a, x)
        subfield_inter(a, a).contains(x) = a.contains(x)
    }
    subfield_ext(subfield_inter(a, a), a)
}

/// Intersection equals the left operand exactly when the left is contained in the right.
theorem subfield_inter_eq_left_iff_le[F: Field](a: Subfield[F], b: Subfield[F]) {
    (subfield_inter(a, b) = a) = subfield_le(a, b)
} by {
    if subfield_inter(a, b) = a {
        subfield_inter_le_right(a, b)
        subfield_le(subfield_inter(a, b), b)
        subfield_le(a, b)
    }
    if subfield_le(a, b) {
        subfield_inter_le_left(a, b)
        subfield_le(subfield_inter(a, b), a)
        subfield_le_refl(a)
        subfield_inter_greatest(a, b, a)
        subfield_le(a, subfield_inter(a, b))
        subfield_le_antisymm(subfield_inter(a, b), a)
        subfield_inter(a, b) = a
    }
}

/// Subfield intersection is monotone in its left argument.
theorem subfield_inter_mono_left[F: Field](a: Subfield[F], b: Subfield[F], c: Subfield[F]) {
    subfield_le(a, b) implies subfield_le(subfield_inter(a, c), subfield_inter(b, c))
} by {
    if subfield_le(a, b) {
        subfield_le(a, b) = forall(x: F) { a.contains(x) implies b.contains(x) }
        forall(x: F) {
            if subfield_inter(a, c).contains(x) {
                subfield_inter_contains_eq(a, c, x)
                a.contains(x) and c.contains(x)
                b.contains(x)
                subfield_inter_contains_eq(b, c, x)
                subfield_inter(b, c).contains(x)
            }
        }
    }
}

/// Subfield intersection is monotone in its right argument.
theorem subfield_inter_mono_right[F: Field](a: Subfield[F], b: Subfield[F], c: Subfield[F]) {
    subfield_le(a, b) implies subfield_le(subfield_inter(c, a), subfield_inter(c, b))
} by {
    if subfield_le(a, b) {
        subfield_le(a, b) = forall(x: F) { a.contains(x) implies b.contains(x) }
        forall(x: F) {
            if subfield_inter(c, a).contains(x) {
                subfield_inter_contains_eq(c, a, x)
                c.contains(x) and a.contains(x)
                b.contains(x)
                subfield_inter_contains_eq(c, b, x)
                subfield_inter(c, b).contains(x)
            }
        }
    }
}

/// Subfield intersection is monotone in both arguments.
theorem subfield_inter_mono[F: Field](a: Subfield[F], b: Subfield[F], c: Subfield[F], d: Subfield[F]) {
    subfield_le(a, b) and subfield_le(c, d) implies subfield_le(subfield_inter(a, c), subfield_inter(b, d))
} by {
    if subfield_le(a, b) and subfield_le(c, d) {
        subfield_le(a, b) = forall(x: F) { a.contains(x) implies b.contains(x) }
        subfield_le(c, d) = forall(x: F) { c.contains(x) implies d.contains(x) }
        forall(x: F) {
            if subfield_inter(a, c).contains(x) {
                subfield_inter_contains_eq(a, c, x)
                a.contains(x) and c.contains(x)
                b.contains(x)
                d.contains(x)
                subfield_inter_contains_eq(b, d, x)
                subfield_inter(b, d).contains(x)
            }
        }
    }
}

/// A subfield is contained in an intersection exactly when it is contained in both operands.
theorem subfield_le_inter_iff[F: Field](a: Subfield[F], b: Subfield[F], c: Subfield[F]) {
    subfield_le(c, subfield_inter(a, b)) = (subfield_le(c, a) and subfield_le(c, b))
} by {
    if subfield_le(c, subfield_inter(a, b)) {
        subfield_inter_le_left(a, b)
        subfield_le_trans(c, subfield_inter(a, b), a)
        subfield_inter_le_right(a, b)
        subfield_le_trans(c, subfield_inter(a, b), b)
        subfield_le(c, a) and subfield_le(c, b)
    }
    if subfield_le(c, a) and subfield_le(c, b) {
        subfield_inter_greatest(a, b, c)
        subfield_le(c, subfield_inter(a, b))
    }
}

/// Subfield intersection is associative.
theorem subfield_inter_assoc[F: Field](a: Subfield[F], b: Subfield[F], c: Subfield[F]) {
    subfield_inter(subfield_inter(a, b), c) = subfield_inter(a, subfield_inter(b, c))
} by {
    forall(x: F) {
        subfield_inter_contains_eq(subfield_inter(a, b), c, x)
        subfield_inter(subfield_inter(a, b), c).contains(x) = (subfield_inter(a, b).contains(x) and c.contains(x))
        subfield_inter_contains_eq(a, b, x)
        subfield_inter(a, b).contains(x) = (a.contains(x) and b.contains(x))
        subfield_inter(subfield_inter(a, b), c).contains(x) = (a.contains(x) and b.contains(x) and c.contains(x))
        subfield_inter_contains_eq(a, subfield_inter(b, c), x)
        subfield_inter(a, subfield_inter(b, c)).contains(x) = (a.contains(x) and subfield_inter(b, c).contains(x))
        subfield_inter_contains_eq(b, c, x)
        subfield_inter(b, c).contains(x) = (b.contains(x) and c.contains(x))
        if subfield_inter(subfield_inter(a, b), c).contains(x) {
            a.contains(x) and b.contains(x) and c.contains(x)
            subfield_inter(a, subfield_inter(b, c)).contains(x)
        }
        if subfield_inter(a, subfield_inter(b, c)).contains(x) {
            a.contains(x) and b.contains(x) and c.contains(x)
            subfield_inter(subfield_inter(a, b), c).contains(x)
        }
        subfield_inter(subfield_inter(a, b), c).contains(x) = subfield_inter(a, subfield_inter(b, c)).contains(x)
    }
    subfield_ext(subfield_inter(subfield_inter(a, b), c), subfield_inter(a, subfield_inter(b, c)))
}
