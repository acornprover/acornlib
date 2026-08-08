from algebra.add_comm_group import AddCommGroup
from algebra.add_group import AddGroup, AddGroupHom, add_group_hom_zero, add_group_hom_add, add_group_hom_neg
from algebra.add_submonoid import AddSubmonoid, add_submonoid_zero_constraint,
    add_submonoid_closure_constraint, add_submonoid_constraint
from data.basic.functions import predicate_extensionality
from data.basic.set import Set, is_subset_monotone_map, is_set_extensive_map, is_set_idempotent_map,
    is_set_closure_operator, union_contains_eq, union_contains_left, union_contains_right
from data.basic.relation_basic import is_reflexive, is_symmetric, is_transitive, is_equivalence
from data.basic.relation_transport import respects_add, is_add_congruence, respects_binary_op,
    respects_unary_op, is_binary_congruence, respects_add_eq_respects_binary_op,
    add_congruence_eq_binary_congruence
from data.basic.equivalence import QuotientRelation, QuotientOver, quotient_over_mk,
    quotient_relation_new_round_trip, quotient_unary_respects, quotient_binary_respects,
    quotient_unary_respects_eq_respects_unary_op,
    quotient_binary_respects_eq_respects_binary_op,
    quotient_over_binary_op, quotient_over_unary_op,
    quotient_over_binary_op_projection, quotient_over_binary_op_projection_left,
    quotient_over_binary_op_projection_right,
    quotient_over_unary_op_projection, quotient_over_unary_op_projection_compatible

/// True if a subset contains the additive identity.
define add_zero_constraint[G: AddGroup](contains: G -> Bool) -> Bool {
    contains(G.0)
}

/// True if a subset is closed under addition.
define add_closure_constraint[G: AddGroup](contains: G -> Bool) -> Bool {
    forall(a: G, b: G) {
        contains(a) and contains(b) implies contains(a + b)
    }
}

/// True if a subset is closed under additive negation.
define neg_constraint[G: AddGroup](contains: G -> Bool) -> Bool {
    forall(a: G) {
        contains(a) implies contains(-a)
    }
}

/// True if a subset satisfies all the requirements to be an additive subgroup.
define add_subgroup_constraint[G: AddGroup](contains: G -> Bool) -> Bool {
    add_zero_constraint(contains) and add_closure_constraint(contains) and neg_constraint(contains)
}

/// An additive subgroup of an additive group `G`, represented as a subset closed under the additive group operations.
structure AddSubgroup[G: AddGroup] {
    /// True if the given element is a member of this additive subgroup.
    contains: G -> Bool
} constraint {
    add_subgroup_constraint(contains)
}

/// Additive subgroup extensionality from equality of membership.
theorem add_subgroup_ext[G: AddGroup](a: AddSubgroup[G], b: AddSubgroup[G]) {
    (forall(x: G) { a.contains(x) = b.contains(x) }) implies a = b
} by {
    if forall(x: G) { a.contains(x) = b.contains(x) } {
        a.contains = b.contains
    }
}

/// Equal additive subgroups have equal membership predicates.
theorem add_subgroup_eq_contains[G: AddGroup](a: AddSubgroup[G], b: AddSubgroup[G]) {
    a = b implies a.contains = b.contains
}

/// Equal additive subgroups have equal membership at every element.
theorem add_subgroup_eq_contains_at[G: AddGroup](a: AddSubgroup[G], b: AddSubgroup[G], x: G) {
    a = b implies a.contains(x) = b.contains(x)
} by {
    if a = b {
        a.contains(x) = b.contains(x)
    }
}

/// Equality of additive subgroups transports predicates on additive subgroups.
theorem add_subgroup_eq_transport_predicate[G: AddGroup](
    p: AddSubgroup[G] -> Bool,
    a: AddSubgroup[G],
    b: AddSubgroup[G]
) {
    a = b and p(a) implies p(b)
}

/// Equality of additive subgroups transports predicates on additive subgroups in the reverse direction.
theorem add_subgroup_eq_transport_predicate_rev[G: AddGroup](
    p: AddSubgroup[G] -> Bool,
    a: AddSubgroup[G],
    b: AddSubgroup[G]
) {
    a = b and p(b) implies p(a)
}

/// Equality of membership predicates determines equality of additive subgroups.
theorem add_subgroup_eq_of_contains_eq[G: AddGroup](a: AddSubgroup[G], b: AddSubgroup[G]) {
    a.contains = b.contains implies a = b
} by {
    if a.contains = b.contains {
        forall(x: G) {
            a.contains(x) = b.contains(x)
        }
        add_subgroup_ext(a, b)
    }
}

/// Pointwise equality of membership determines equality of additive subgroups.
theorem add_subgroup_eq_of_contains_at_eq[G: AddGroup](a: AddSubgroup[G], b: AddSubgroup[G]) {
    (forall(x: G) { a.contains(x) = b.contains(x) }) implies a = b
} by {
    if forall(x: G) { a.contains(x) = b.contains(x) } {
        add_subgroup_ext(a, b)
    }
}

/// Membership of zero in any additive subgroup.
theorem add_subgroup_contains_zero[G: AddGroup](h: AddSubgroup[G]) {
    h.contains(G.0)
} by {
    add_subgroup_constraint(h.contains)
    add_subgroup_constraint(h.contains) =
        add_zero_constraint(h.contains) and add_closure_constraint(h.contains) and neg_constraint(h.contains)
}

/// Closure of any additive subgroup under addition.
theorem add_subgroup_contains_add[G: AddGroup](h: AddSubgroup[G], a: G, b: G) {
    h.contains(a) and h.contains(b) implies h.contains(a + b)
} by {
    if h.contains(a) and h.contains(b) {
        add_subgroup_constraint(h.contains)
        add_subgroup_constraint(h.contains) =
            add_zero_constraint(h.contains) and add_closure_constraint(h.contains) and neg_constraint(h.contains)
        add_closure_constraint(h.contains) = forall(x: G, y: G) {
            h.contains(x) and h.contains(y) implies h.contains(x + y)
        }
        h.contains(a + b)
    }
}

/// Closure of any additive subgroup under negation.
theorem add_subgroup_contains_neg[G: AddGroup](h: AddSubgroup[G], a: G) {
    h.contains(a) implies h.contains(-a)
} by {
    if h.contains(a) {
        neg_constraint(h.contains)
        neg_constraint(h.contains) = forall(x: G) {
            h.contains(x) implies h.contains(-x)
        }
        h.contains(-a)
    }
}

/// Membership is unchanged by additive negation.
theorem add_subgroup_contains_neg_iff[G: AddGroup](h: AddSubgroup[G], a: G) {
    h.contains(-a) = h.contains(a)
} by {
    if h.contains(-a) {
        add_subgroup_contains_neg(h, -a)
        --a = a
        h.contains(a)
    }
    if h.contains(a) {
        add_subgroup_contains_neg(h, a)
        h.contains(-a)
    }
}

/// Closure of any additive subgroup under subtraction.
theorem add_subgroup_contains_sub[G: AddGroup](h: AddSubgroup[G], a: G, b: G) {
    h.contains(a) and h.contains(b) implies h.contains(a - b)
} by {
    if h.contains(a) and h.contains(b) {
        add_subgroup_contains_neg(h, b)
        add_subgroup_contains_add(h, a, -b)
        a - b = a + -b
    }
}

/// Closure of any additive subgroup under sums of three elements.
theorem add_subgroup_contains_add_add[G: AddGroup](h: AddSubgroup[G], a: G, b: G, c: G) {
    h.contains(a) and h.contains(b) and h.contains(c) implies h.contains(a + b + c)
} by {
    if h.contains(a) and h.contains(b) and h.contains(c) {
        add_subgroup_contains_add(h, a, b)
        add_subgroup_contains_add(h, a + b, c)
        h.contains(a + b + c)
    }
}

/// Closure of any additive subgroup under a negated element added on the left.
theorem add_subgroup_contains_neg_add[G: AddGroup](h: AddSubgroup[G], a: G, b: G) {
    h.contains(a) and h.contains(b) implies h.contains(-a + b)
} by {
    if h.contains(a) and h.contains(b) {
        add_subgroup_contains_neg(h, a)
        add_subgroup_contains_add(h, -a, b)
        h.contains(-a + b)
    }
}

/// The common membership predicate of two additive subgroups.
define add_subgroup_intersection_contains[G: AddGroup](a: AddSubgroup[G], b: AddSubgroup[G],
    x: G) -> Bool {
    a.contains(x) and b.contains(x)
}

/// The common part of two additive subgroups contains zero.
theorem add_subgroup_intersection_zero_constraint[G: AddGroup](a: AddSubgroup[G], b: AddSubgroup[G]) {
    add_zero_constraint(add_subgroup_intersection_contains(a, b))
} by {
    add_subgroup_contains_zero(a)
    add_subgroup_contains_zero(b)
}

/// The common part of two additive subgroups is closed under addition.
theorem add_subgroup_intersection_closure_constraint[G: AddGroup](a: AddSubgroup[G], b: AddSubgroup[G]) {
    add_closure_constraint(add_subgroup_intersection_contains(a, b))
} by {
    forall(x: G, y: G) {
        if add_subgroup_intersection_contains(a, b, x) and add_subgroup_intersection_contains(a, b, y) {
            a.contains(y)
            b.contains(y)
            add_subgroup_contains_add(a, x, y)
            add_subgroup_contains_add(b, x, y)
            b.contains(x + y)
            add_subgroup_intersection_contains(a, b, x + y)
        }
    }
}

/// The common part of two additive subgroups is closed under negation.
theorem add_subgroup_intersection_neg_constraint[G: AddGroup](a: AddSubgroup[G], b: AddSubgroup[G]) {
    neg_constraint(add_subgroup_intersection_contains(a, b))
} by {
    forall(x: G) {
        if add_subgroup_intersection_contains(a, b, x) {
            add_subgroup_contains_neg(a, x)
            add_subgroup_contains_neg(b, x)
            add_subgroup_intersection_contains(a, b, -x)
        }
    }
}

/// The common part of two additive subgroups is an additive subgroup.
theorem add_subgroup_intersection_constraint[G: AddGroup](a: AddSubgroup[G], b: AddSubgroup[G]) {
    add_subgroup_constraint(add_subgroup_intersection_contains(a, b))
} by {
    add_subgroup_intersection_zero_constraint(a, b)
    add_subgroup_intersection_closure_constraint(a, b)
    add_subgroup_intersection_neg_constraint(a, b)
}

let add_subgroup_intersection[G: AddGroup](a: AddSubgroup[G], b: AddSubgroup[G]) -> result: AddSubgroup[G] satisfy {
    AddSubgroup.new(add_subgroup_intersection_contains(a, b)) = Option.some(result)
} by {
    add_subgroup_intersection_constraint(a, b)
}

attributes AddSubgroup[G: AddGroup] {
    /// Additive subgroup extensionality from pointwise equality of membership.
    let ext = add_subgroup_ext[G]

    /// The subset of additive group elements belonging to this additive subgroup.
    define as_set(self) -> Set[G] {
        Set[G].new(self.contains)
    }

    /// The common part of two additive subgroups.
    let intersection: (AddSubgroup[G], AddSubgroup[G]) -> AddSubgroup[G] = add_subgroup_intersection
}

/// Membership in the underlying set is membership in the additive subgroup.
theorem add_subgroup_as_set_contains_eq[G: AddGroup](s: AddSubgroup[G], x: G) {
    s.as_set.contains(x) = s.contains(x)
} by {
}

/// Membership in an additive subgroup is membership in its underlying set.
theorem add_subgroup_contains_as_set_eq[G: AddGroup](s: AddSubgroup[G], x: G) {
    s.contains(x) = s.as_set.contains(x)
} by {
    add_subgroup_as_set_contains_eq(s, x)
}

/// Equal additive subgroups have equal underlying sets.
theorem add_subgroup_eq_as_set[G: AddGroup](a: AddSubgroup[G], b: AddSubgroup[G]) {
    a = b implies a.as_set = b.as_set
}

/// Membership in the intersection means membership in both additive subgroups.
theorem add_subgroup_intersection_contains_eq[G: AddGroup](a: AddSubgroup[G], b: AddSubgroup[G],
    x: G) {
    a.intersection(b).contains(x) = (a.contains(x) and b.contains(x))
} by {
    a.intersection(b).contains(x) = add_subgroup_intersection_contains(a, b, x)
}

/// The intersection is contained in the left additive subgroup.
theorem add_subgroup_intersection_subset_left[G: AddGroup](a: AddSubgroup[G], b: AddSubgroup[G],
    x: G) {
    a.intersection(b).contains(x) implies a.contains(x)
} by {
    if a.intersection(b).contains(x) {
        add_subgroup_intersection_contains_eq(a, b, x)
        a.contains(x)
    }
}

/// The intersection is contained in the right additive subgroup.
theorem add_subgroup_intersection_subset_right[G: AddGroup](a: AddSubgroup[G], b: AddSubgroup[G],
    x: G) {
    a.intersection(b).contains(x) implies b.contains(x)
} by {
    if a.intersection(b).contains(x) {
        add_subgroup_intersection_contains_eq(a, b, x)
        b.contains(x)
    }
}

/// Additive subgroup intersection is commutative.
theorem add_subgroup_intersection_comm[G: AddGroup](a: AddSubgroup[G], b: AddSubgroup[G]) {
    a.intersection(b) = b.intersection(a)
} by {
    forall(x: G) {
        if a.intersection(b).contains(x) {
            add_subgroup_intersection_contains_eq(a, b, x)
            add_subgroup_intersection_contains_eq(b, a, x)
            b.intersection(a).contains(x)
        }
        if b.intersection(a).contains(x) {
            add_subgroup_intersection_contains_eq(b, a, x)
            add_subgroup_intersection_contains_eq(a, b, x)
            a.intersection(b).contains(x)
        }
        a.intersection(b).contains(x) = b.intersection(a).contains(x)
    }
    add_subgroup_ext(a.intersection(b), b.intersection(a))
}

/// Additive subgroup intersection is associative.
theorem add_subgroup_intersection_assoc[G: AddGroup](
    a: AddSubgroup[G],
    b: AddSubgroup[G],
    c: AddSubgroup[G]
) {
    a.intersection(b).intersection(c) = a.intersection(b.intersection(c))
} by {
    let lhs = a.intersection(b).intersection(c)
    let rhs = a.intersection(b.intersection(c))
    forall(x: G) {
        if lhs.contains(x) {
            add_subgroup_intersection_contains_eq(a.intersection(b), c, x)
            add_subgroup_intersection_contains_eq(a, b, x)
            b.contains(x)
            add_subgroup_intersection_contains_eq(b, c, x)
            b.intersection(c).contains(x)
            add_subgroup_intersection_contains_eq(a, b.intersection(c), x)
            rhs.contains(x)
        }
        if rhs.contains(x) {
            add_subgroup_intersection_contains_eq(a, b.intersection(c), x)
            add_subgroup_intersection_contains_eq(b, c, x)
            b.contains(x)
            add_subgroup_intersection_contains_eq(a, b, x)
            a.intersection(b).contains(x)
            add_subgroup_intersection_contains_eq(a.intersection(b), c, x)
            lhs.contains(x)
        }
        lhs.contains(x) = rhs.contains(x)
    }
    add_subgroup_ext(lhs, rhs)
}

/// Additive subgroup intersection is idempotent.
theorem add_subgroup_intersection_idempotent[G: AddGroup](a: AddSubgroup[G]) {
    a.intersection(a) = a
} by {
    forall(x: G) {
        if a.intersection(a).contains(x) {
            add_subgroup_intersection_contains_eq(a, a, x)
            a.contains(x)
        }
        if a.contains(x) {
            add_subgroup_intersection_contains_eq(a, a, x)
            a.intersection(a).contains(x)
        }
        a.intersection(a).contains(x) = a.contains(x)
    }
    add_subgroup_ext(a.intersection(a), a)
}

/// True if every element of one additive subgroup belongs to another.
define add_subgroup_subset[G: AddGroup](a: AddSubgroup[G], b: AddSubgroup[G]) -> Bool {
    forall(x: G) {
        a.contains(x) implies b.contains(x)
    }
}

/// Additive subgroup containment is containment of the underlying sets.
theorem add_subgroup_subset_as_set_eq[G: AddGroup](a: AddSubgroup[G], b: AddSubgroup[G]) {
    add_subgroup_subset(a, b) = a.as_set.subset(b.as_set)
} by {
    if add_subgroup_subset(a, b) {
        forall(x: G) {
            if a.as_set.contains(x) {
                add_subgroup_as_set_contains_eq(a, x)
                a.contains(x)
                b.contains(x)
                add_subgroup_as_set_contains_eq(b, x)
                b.as_set.contains(x)
            }
        }
        a.as_set.subset(b.as_set)
    }
    if a.as_set.subset(b.as_set) {
        forall(x: G) {
            if a.contains(x) {
                add_subgroup_as_set_contains_eq(a, x)
                a.as_set.subset(b.as_set) = forall(y: G) {
                    a.as_set.contains(y) implies b.as_set.contains(y)
                }
                add_subgroup_as_set_contains_eq(b, x)
                b.contains(x)
            }
        }
        add_subgroup_subset(a, b)
    }
}

/// Additive subgroup containment gives containment of underlying sets.
theorem add_subgroup_as_set_subset_of_subset[G: AddGroup](a: AddSubgroup[G], b: AddSubgroup[G]) {
    add_subgroup_subset(a, b) implies a.as_set.subset(b.as_set)
} by {
    if add_subgroup_subset(a, b) {
        add_subgroup_subset_as_set_eq(a, b)
        a.as_set.subset(b.as_set)
    }
}

/// Containment of underlying sets gives additive subgroup containment.
theorem add_subgroup_subset_of_as_set_subset[G: AddGroup](a: AddSubgroup[G], b: AddSubgroup[G]) {
    a.as_set.subset(b.as_set) implies add_subgroup_subset(a, b)
} by {
    if a.as_set.subset(b.as_set) {
        add_subgroup_subset_as_set_eq(a, b)
        add_subgroup_subset(a, b)
    }
}

/// Additive subgroup inclusion is reflexive.
theorem add_subgroup_subset_refl[G: AddGroup](a: AddSubgroup[G]) {
    add_subgroup_subset(a, a)
} by {
    forall(x: G) {
        a.contains(x) implies a.contains(x)
    }
}

/// Additive subgroup inclusion is transitive.
theorem add_subgroup_subset_trans[G: AddGroup](
    a: AddSubgroup[G],
    b: AddSubgroup[G],
    c: AddSubgroup[G]
) {
    add_subgroup_subset(a, b) and add_subgroup_subset(b, c) implies add_subgroup_subset(a, c)
} by {
    if add_subgroup_subset(a, b) and add_subgroup_subset(b, c) {
        forall(x: G) {
            if a.contains(x) {
                b.contains(x)
                c.contains(x)
            }
        }
    }
}

/// The intersection of two additive subgroups is contained in the left additive subgroup.
theorem add_subgroup_intersection_subset_left_relation[G: AddGroup](
    a: AddSubgroup[G],
    b: AddSubgroup[G]
) {
    add_subgroup_subset(a.intersection(b), a)
} by {
    forall(x: G) {
        if a.intersection(b).contains(x) {
            add_subgroup_intersection_contains_eq(a, b, x)
            a.contains(x)
        }
    }
}

/// The intersection of two additive subgroups is contained in the right additive subgroup.
theorem add_subgroup_intersection_subset_right_relation[G: AddGroup](
    a: AddSubgroup[G],
    b: AddSubgroup[G]
) {
    add_subgroup_subset(a.intersection(b), b)
} by {
    forall(x: G) {
        if a.intersection(b).contains(x) {
            add_subgroup_intersection_contains_eq(a, b, x)
            b.contains(x)
        }
    }
}

/// Every common additive subgroup of two additive subgroups is contained in their intersection.
theorem add_subgroup_subset_intersection_of_subset_left_right[G: AddGroup](
    c: AddSubgroup[G],
    a: AddSubgroup[G],
    b: AddSubgroup[G]
) {
    add_subgroup_subset(c, a) and add_subgroup_subset(c, b) implies add_subgroup_subset(c, a.intersection(b))
} by {
    if add_subgroup_subset(c, a) and add_subgroup_subset(c, b) {
        forall(x: G) {
            if c.contains(x) {
                a.contains(x)
                b.contains(x)
                add_subgroup_intersection_contains_eq(a, b, x)
                a.intersection(b).contains(x)
            }
        }
    }
}

/// Containment in an intersection is equivalent to containment in both additive subgroups.
theorem add_subgroup_subset_intersection_iff[G: AddGroup](
    c: AddSubgroup[G],
    a: AddSubgroup[G],
    b: AddSubgroup[G]
) {
    add_subgroup_subset(c, a.intersection(b)) = (add_subgroup_subset(c, a) and add_subgroup_subset(c, b))
} by {
    if add_subgroup_subset(c, a.intersection(b)) {
        add_subgroup_intersection_subset_left_relation(a, b)
        add_subgroup_subset_trans(c, a.intersection(b), a)
        add_subgroup_intersection_subset_right_relation(a, b)
        add_subgroup_subset_trans(c, a.intersection(b), b)
        add_subgroup_subset(c, b)
        add_subgroup_subset(c, a) and add_subgroup_subset(c, b)
    }
    if add_subgroup_subset(c, a) and add_subgroup_subset(c, b) {
        add_subgroup_subset_intersection_of_subset_left_right(c, a, b)
        add_subgroup_subset(c, a.intersection(b))
    }
}

/// True if an element is the additive identity.
define is_add_identity[G: AddGroup](g: G) -> Bool {
    g = G.0
}

/// The additive identity predicate is an additive subgroup predicate.
theorem zero_add_subgroup_constraint[G: AddGroup] {
    add_subgroup_constraint(is_add_identity[G])
} by {
    forall(a: G, b: G) {
        if is_add_identity(a) and is_add_identity(b) {
            b = G.0
            is_add_identity(a + b)
        }
    }
    add_closure_constraint(is_add_identity[G])
    forall(a: G) {
        if is_add_identity(a) {
            a = G.0
            -a = G.0
            is_add_identity(-a)
        }
    }
    neg_constraint(is_add_identity[G])
}

/// The additive subgroup containing only the additive identity.
let zero_add_subgroup[G: AddGroup]: AddSubgroup[G] satisfy {
    AddSubgroup.new(is_add_identity[G]) = Option.some(zero_add_subgroup)
}

/// Only the additive identity belongs to the zero additive subgroup.
theorem zero_add_subgroup_only_has_zero[G: AddGroup](g: G) {
    zero_add_subgroup[G].contains(g) implies g = G.0
} by {
    if zero_add_subgroup[G].contains(g) {
        zero_add_subgroup[G].contains(g) = is_add_identity(g)
        g = G.0
    }
}

/// Membership in the zero additive subgroup is the same as equality to zero.
theorem zero_add_subgroup_contains_eq[G: AddGroup](g: G) {
    zero_add_subgroup[G].contains(g) = (g = G.0)
} by {
    if zero_add_subgroup[G].contains(g) {
        zero_add_subgroup_only_has_zero(g)
        g = G.0
    }
    if g = G.0 {
        add_subgroup_contains_zero(zero_add_subgroup[G])
        zero_add_subgroup[G].contains(g)
    }
}

/// The zero additive subgroup is contained in every additive subgroup.
theorem zero_add_subgroup_subset[G: AddGroup](s: AddSubgroup[G]) {
    add_subgroup_subset(zero_add_subgroup[G], s)
} by {
    forall(x: G) {
        if zero_add_subgroup[G].contains(x) {
            zero_add_subgroup_only_has_zero(x)
            add_subgroup_contains_zero(s)
            s.contains(x)
        }
    }
}

/// The full subset of any additive group is an additive subgroup.
define full_add_group_contains[G: AddGroup](a: G) -> Bool {
    true
}

/// The full membership predicate is an additive subgroup predicate.
theorem full_add_subgroup_constraint[G: AddGroup] {
    add_subgroup_constraint(full_add_group_contains[G])
} by {
    forall(a: G, b: G) {
        if full_add_group_contains[G](a) and full_add_group_contains[G](b) {
            full_add_group_contains[G](a + b)
        }
    }
    add_closure_constraint(full_add_group_contains[G])
    forall(a: G) {
        if full_add_group_contains[G](a) {
            full_add_group_contains[G](-a)
        }
    }
    neg_constraint(full_add_group_contains[G])
}

/// The full additive subgroup, containing every element.
let full_add_subgroup[G: AddGroup]: AddSubgroup[G] satisfy {
    AddSubgroup.new(full_add_group_contains[G]) = Option.some(full_add_subgroup)
}

/// Every element belongs to the full additive subgroup.
theorem full_add_subgroup_contains_everything[G: AddGroup](g: G) {
    full_add_subgroup[G].contains(g)
}

/// Membership in the full additive subgroup is always true.
theorem full_add_subgroup_contains_eq[G: AddGroup](g: G) {
    full_add_subgroup[G].contains(g) = true
} by {
    full_add_subgroup_contains_everything(g)
}

/// Every additive subgroup is contained in the full additive subgroup.
theorem add_subgroup_subset_full[G: AddGroup](s: AddSubgroup[G]) {
    add_subgroup_subset(s, full_add_subgroup[G])
} by {
    forall(x: G) {
        if s.contains(x) {
            full_add_subgroup_contains_everything(x)
        }
    }
}

/// Mutual inclusion of additive subgroups forces equality.
theorem add_subgroup_subset_antisymm[G: AddGroup](a: AddSubgroup[G], b: AddSubgroup[G]) {
    add_subgroup_subset(a, b) and add_subgroup_subset(b, a) implies a = b
} by {
    if add_subgroup_subset(a, b) and add_subgroup_subset(b, a) {
        add_subgroup_subset(a, b) = forall(y: G) {
            a.contains(y) implies b.contains(y)
        }
        add_subgroup_subset(b, a) = forall(y: G) {
            b.contains(y) implies a.contains(y)
        }
        forall(x: G) {
            if a.contains(x) {
                b.contains(x)
            }
            if b.contains(x) {
                a.contains(x)
            }
            a.contains(x) = b.contains(x)
        }
        predicate_extensionality(a.contains, b.contains)
        a = b
    }
}

/// Equality of additive subgroups is equivalent to mutual inclusion.
theorem add_subgroup_eq_iff_subset_both[G: AddGroup](a: AddSubgroup[G], b: AddSubgroup[G]) {
    a = b = (add_subgroup_subset(a, b) and add_subgroup_subset(b, a))
} by {
    if a = b {
        add_subgroup_subset_refl(a)
        add_subgroup_subset(b, a)
        add_subgroup_subset(a, b) and add_subgroup_subset(b, a)
    }
    if add_subgroup_subset(a, b) and add_subgroup_subset(b, a) {
        add_subgroup_subset_antisymm(a, b)
        a = b
    }
}

/// Intersecting with a larger additive subgroup gives the smaller additive subgroup.
theorem add_subgroup_intersection_eq_left_of_subset[G: AddGroup](
    a: AddSubgroup[G],
    b: AddSubgroup[G]
) {
    add_subgroup_subset(a, b) implies a.intersection(b) = a
} by {
    if add_subgroup_subset(a, b) {
        add_subgroup_intersection_subset_left_relation(a, b)
        add_subgroup_subset_intersection_of_subset_left_right(a, a, b)
        add_subgroup_subset(a, a.intersection(b))
        add_subgroup_subset_antisymm(a.intersection(b), a)
        a.intersection(b) = a
    }
}

/// Intersecting with a larger additive subgroup on the left gives the smaller additive subgroup.
theorem add_subgroup_intersection_eq_right_of_subset[G: AddGroup](
    a: AddSubgroup[G],
    b: AddSubgroup[G]
) {
    add_subgroup_subset(b, a) implies a.intersection(b) = b
} by {
    if add_subgroup_subset(b, a) {
        add_subgroup_intersection_subset_right_relation(a, b)
        add_subgroup_subset_intersection_of_subset_left_right(b, a, b)
        add_subgroup_subset_antisymm(a.intersection(b), b)
        a.intersection(b) = b
    }
}

/// Intersecting the zero additive subgroup on the left gives the zero additive subgroup.
theorem zero_add_subgroup_intersection_left[G: AddGroup](s: AddSubgroup[G]) {
    zero_add_subgroup[G].intersection(s) = zero_add_subgroup[G]
} by {
    zero_add_subgroup_subset(s)
    add_subgroup_intersection_eq_left_of_subset(zero_add_subgroup[G], s)
}

/// Intersecting the zero additive subgroup on the right gives the zero additive subgroup.
theorem zero_add_subgroup_intersection_right[G: AddGroup](s: AddSubgroup[G]) {
    s.intersection(zero_add_subgroup[G]) = zero_add_subgroup[G]
} by {
    zero_add_subgroup_subset(s)
    add_subgroup_intersection_eq_right_of_subset(s, zero_add_subgroup[G])
}

/// Intersecting the full additive subgroup on the left gives the other additive subgroup.
theorem full_add_subgroup_intersection_left[G: AddGroup](s: AddSubgroup[G]) {
    full_add_subgroup[G].intersection(s) = s
} by {
    add_subgroup_subset_full(s)
    add_subgroup_intersection_eq_right_of_subset(full_add_subgroup[G], s)
}

/// Intersecting the full additive subgroup on the right gives the other additive subgroup.
theorem full_add_subgroup_intersection_right[G: AddGroup](s: AddSubgroup[G]) {
    s.intersection(full_add_subgroup[G]) = s
} by {
    add_subgroup_subset_full(s)
    add_subgroup_intersection_eq_left_of_subset(s, full_add_subgroup[G])
}

/// True if every element of a set belongs to an additive subgroup.
define set_subset_add_subgroup[G: AddGroup](a: Set[G], s: AddSubgroup[G]) -> Bool {
    forall(x: G) {
        a.contains(x) implies s.contains(x)
    }
}

/// Set containment in an additive subgroup is containment in its underlying set.
theorem set_subset_add_subgroup_as_set_eq[G: AddGroup](a: Set[G], s: AddSubgroup[G]) {
    set_subset_add_subgroup(a, s) = a.subset(s.as_set)
} by {
    if set_subset_add_subgroup(a, s) {
        forall(x: G) {
            if a.contains(x) {
                s.contains(x)
                add_subgroup_as_set_contains_eq(s, x)
                s.as_set.contains(x)
            }
        }
        a.subset(s.as_set)
    }
    if a.subset(s.as_set) {
        forall(x: G) {
            if a.contains(x) {
                a.subset(s.as_set) = forall(y: G) {
                    a.contains(y) implies s.as_set.contains(y)
                }
                add_subgroup_as_set_contains_eq(s, x)
                s.contains(x)
            }
        }
        set_subset_add_subgroup(a, s)
    }
}

/// Set containment in an additive subgroup gives containment in its underlying set.
theorem set_as_set_subset_of_subset_add_subgroup[G: AddGroup](a: Set[G], s: AddSubgroup[G]) {
    set_subset_add_subgroup(a, s) implies a.subset(s.as_set)
} by {
    if set_subset_add_subgroup(a, s) {
        set_subset_add_subgroup_as_set_eq(a, s)
        a.subset(s.as_set)
    }
}

/// Containment in the underlying set gives set containment in the additive subgroup.
theorem set_subset_add_subgroup_of_as_set_subset[G: AddGroup](a: Set[G], s: AddSubgroup[G]) {
    a.subset(s.as_set) implies set_subset_add_subgroup(a, s)
} by {
    if a.subset(s.as_set) {
        set_subset_add_subgroup_as_set_eq(a, s)
        set_subset_add_subgroup(a, s)
    }
}

/// The underlying set of an additive subgroup is contained in the additive subgroup.
theorem add_subgroup_as_set_subset_self[G: AddGroup](s: AddSubgroup[G]) {
    set_subset_add_subgroup(s.as_set, s)
} by {
    forall(x: G) {
        if s.as_set.contains(x) {
            s.contains(x)
        }
    }
}

/// Set containment is preserved by enlarging the target additive subgroup.
theorem set_subset_add_subgroup_trans[G: AddGroup](
    a: Set[G],
    s: AddSubgroup[G],
    t: AddSubgroup[G]
) {
    set_subset_add_subgroup(a, s) and add_subgroup_subset(s, t) implies set_subset_add_subgroup(a, t)
} by {
    if set_subset_add_subgroup(a, s) and add_subgroup_subset(s, t) {
        forall(x: G) {
            if a.contains(x) {
                s.contains(x)
                t.contains(x)
            }
        }
    }
}

/// Set containment in an additive subgroup is preserved by enlarging the set.
theorem set_subset_add_subgroup_of_set_subset[G: AddGroup](
    a: Set[G],
    b: Set[G],
    s: AddSubgroup[G]
) {
    a.subset(b) and set_subset_add_subgroup(b, s) implies set_subset_add_subgroup(a, s)
} by {
    if a.subset(b) and set_subset_add_subgroup(b, s) {
        forall(x: G) {
            if a.contains(x) {
                a.subset(b) = forall(y: G) {
                    a.contains(y) implies b.contains(y)
                }
                b.contains(x)
                s.contains(x)
            }
        }
    }
}

/// True if an element belongs to every additive subgroup that contains a given set.
define add_subgroup_closure_contains[G: AddGroup](a: Set[G], x: G) -> Bool {
    forall(s: AddSubgroup[G]) {
        set_subset_add_subgroup(a, s) implies s.contains(x)
    }
}

/// Membership in the closure gives membership in each additive subgroup that contains the set.
theorem add_subgroup_closure_contains_of_set_subset_raw[G: AddGroup](
    a: Set[G],
    s: AddSubgroup[G],
    x: G
) {
    add_subgroup_closure_contains(a, x) and set_subset_add_subgroup(a, s) implies s.contains(x)
} by {
    if add_subgroup_closure_contains(a, x) and set_subset_add_subgroup(a, s) {
        add_subgroup_closure_contains(a, x) = forall(t: AddSubgroup[G]) {
            set_subset_add_subgroup(a, t) implies t.contains(x)
        }
        s.contains(x)
    }
}

/// The additive subgroup closure of a set contains zero.
theorem add_subgroup_closure_zero_constraint[G: AddGroup](a: Set[G]) {
    add_zero_constraint(add_subgroup_closure_contains(a))
} by {
    forall(s: AddSubgroup[G]) {
        if set_subset_add_subgroup(a, s) {
            add_subgroup_contains_zero(s)
            s.contains(G.0)
        }
    }
    add_subgroup_closure_contains(a, G.0)
}

/// The additive subgroup closure of a set is closed under addition.
theorem add_subgroup_closure_closure_constraint[G: AddGroup](a: Set[G]) {
    add_closure_constraint(add_subgroup_closure_contains(a))
} by {
    forall(x: G, y: G) {
        if add_subgroup_closure_contains(a, x) and add_subgroup_closure_contains(a, y) {
            forall(s: AddSubgroup[G]) {
                if set_subset_add_subgroup(a, s) {
                    add_subgroup_closure_contains_of_set_subset_raw(a, s, x)
                    add_subgroup_closure_contains_of_set_subset_raw(a, s, y)
                    s.contains(y)
                    add_subgroup_contains_add(s, x, y)
                    s.contains(x + y)
                }
            }
            add_subgroup_closure_contains(a, x + y)
        }
    }
}

/// The additive subgroup closure of a set is closed under negation.
theorem add_subgroup_closure_neg_constraint[G: AddGroup](a: Set[G]) {
    neg_constraint(add_subgroup_closure_contains(a))
} by {
    forall(x: G) {
        if add_subgroup_closure_contains(a, x) {
            forall(s: AddSubgroup[G]) {
                if set_subset_add_subgroup(a, s) {
                    add_subgroup_closure_contains_of_set_subset_raw(a, s, x)
                    add_subgroup_contains_neg(s, x)
                    s.contains(-x)
                }
            }
            add_subgroup_closure_contains(a, -x)
        }
    }
}

/// The additive subgroup generated by a set satisfies the additive subgroup laws.
theorem add_subgroup_generated_constraint[G: AddGroup](a: Set[G]) {
    add_subgroup_constraint(add_subgroup_closure_contains(a))
} by {
    add_subgroup_closure_zero_constraint(a)
    add_subgroup_closure_closure_constraint(a)
    add_subgroup_closure_neg_constraint(a)
}

/// The smallest additive subgroup containing a set.
let add_subgroup_closure[G: AddGroup](a: Set[G]) -> result: AddSubgroup[G] satisfy {
    AddSubgroup.new(add_subgroup_closure_contains(a)) = Option.some(result)
} by {
    add_subgroup_generated_constraint(a)
}

/// Membership in the closure means membership in every additive subgroup containing the set.
theorem add_subgroup_closure_contains_eq[G: AddGroup](a: Set[G], x: G) {
    add_subgroup_closure(a).contains(x) = add_subgroup_closure_contains(a, x)
} by {
}

/// Every generator belongs to the additive subgroup closure.
theorem add_subgroup_subset_closure[G: AddGroup](a: Set[G]) {
    set_subset_add_subgroup(a, add_subgroup_closure(a))
} by {
    forall(x: G) {
        if a.contains(x) {
            forall(s: AddSubgroup[G]) {
                if set_subset_add_subgroup(a, s) {
                    s.contains(x)
                }
            }
            add_subgroup_closure_contains(a, x)
            add_subgroup_closure_contains_eq(a, x)
            add_subgroup_closure(a).contains(x)
        }
    }
}

/// A generator is a member of the additive subgroup closure.
theorem add_subgroup_closure_contains_of_set_contains[G: AddGroup](a: Set[G], x: G) {
    a.contains(x) implies add_subgroup_closure(a).contains(x)
} by {
    if a.contains(x) {
        add_subgroup_subset_closure(a)
        add_subgroup_closure(a).contains(x)
    }
}

/// The additive subgroup closure is contained in any additive subgroup containing the set.
theorem add_subgroup_closure_subset_of_set_subset[G: AddGroup](a: Set[G], s: AddSubgroup[G]) {
    set_subset_add_subgroup(a, s) implies add_subgroup_subset(add_subgroup_closure(a), s)
} by {
    if set_subset_add_subgroup(a, s) {
        forall(x: G) {
            if add_subgroup_closure(a).contains(x) {
                add_subgroup_closure_contains_eq(a, x)
                add_subgroup_closure_contains(a, x)
                add_subgroup_closure_contains_of_set_subset_raw(a, s, x)
            }
        }
    }
}

/// The additive subgroup closure is the least additive subgroup containing the set.
theorem add_subgroup_closure_le_iff_set_subset[G: AddGroup](a: Set[G], s: AddSubgroup[G]) {
    add_subgroup_subset(add_subgroup_closure(a), s) = set_subset_add_subgroup(a, s)
} by {
    if add_subgroup_subset(add_subgroup_closure(a), s) {
        add_subgroup_subset_closure(a)
        set_subset_add_subgroup_trans(a, add_subgroup_closure(a), s)
        set_subset_add_subgroup(a, s)
    }
    if set_subset_add_subgroup(a, s) {
        add_subgroup_closure_subset_of_set_subset(a, s)
        add_subgroup_subset(add_subgroup_closure(a), s)
    }
}

/// The additive subgroup closure and underlying set maps form a set-containment adjunction.
theorem add_subgroup_closure_as_set_galois_connection[G: AddGroup](a: Set[G], s: AddSubgroup[G]) {
    add_subgroup_closure(a).as_set.subset(s.as_set) = a.subset(s.as_set)
} by {
    add_subgroup_subset_as_set_eq(add_subgroup_closure(a), s)
    add_subgroup_closure_le_iff_set_subset(a, s)
    set_subset_add_subgroup_as_set_eq(a, s)
}

/// The closure of the underlying set of an additive subgroup is the additive subgroup itself.
theorem add_subgroup_closure_as_set[G: AddGroup](s: AddSubgroup[G]) {
    add_subgroup_closure(s.as_set) = s
} by {
    add_subgroup_as_set_subset_self(s)
    add_subgroup_closure_subset_of_set_subset(s.as_set, s)
    add_subgroup_subset_closure(s.as_set)
    forall(x: G) {
        if s.contains(x) {
            add_subgroup_closure(s.as_set).contains(x)
        }
    }
    add_subgroup_subset_antisymm(add_subgroup_closure(s.as_set), s)
}

/// Additive subgroup closure is monotone with respect to set inclusion.
theorem add_subgroup_closure_mono[G: AddGroup](a: Set[G], b: Set[G]) {
    a.subset(b) implies add_subgroup_subset(add_subgroup_closure(a), add_subgroup_closure(b))
} by {
    if a.subset(b) {
        add_subgroup_subset_closure(b)
        set_subset_add_subgroup_of_set_subset(a, b, add_subgroup_closure(b))
        add_subgroup_closure_subset_of_set_subset(a, add_subgroup_closure(b))
        add_subgroup_subset(add_subgroup_closure(a), add_subgroup_closure(b))
    }
}

/// Equal sets have equal additive subgroup closures.
theorem add_subgroup_closure_eq_of_set_eq[G: AddGroup](a: Set[G], b: Set[G]) {
    a = b implies add_subgroup_closure(a) = add_subgroup_closure(b)
} by {
    if a = b {
        add_subgroup_closure(a) = add_subgroup_closure(b)
    }
}

/// Applying additive subgroup closure twice gives the same additive subgroup.
theorem add_subgroup_closure_idempotent[G: AddGroup](a: Set[G]) {
    add_subgroup_closure(add_subgroup_closure(a).as_set) = add_subgroup_closure(a)
} by {
    add_subgroup_closure_as_set(add_subgroup_closure(a))
}

/// The set closure induced by additive subgroup generation.
define add_subgroup_set_closure[G: AddGroup](a: Set[G]) -> Set[G] {
    add_subgroup_closure(a).as_set
}

/// The additive subgroup set closure is the underlying set of the generated additive subgroup.
theorem add_subgroup_set_closure_at[G: AddGroup](a: Set[G]) {
    add_subgroup_set_closure(a) = add_subgroup_closure(a).as_set
}

/// The additive subgroup set closure contains the original set.
theorem add_subgroup_set_closure_extensive[G: AddGroup](a: Set[G]) {
    a.subset(add_subgroup_set_closure(a))
} by {
    add_subgroup_set_closure_at(a)
    add_subgroup_subset_closure(a)
    set_subset_add_subgroup_as_set_eq(a, add_subgroup_closure(a))
}

/// The additive subgroup set closure preserves inclusion.
theorem add_subgroup_set_closure_mono[G: AddGroup](a: Set[G], b: Set[G]) {
    a.subset(b) implies add_subgroup_set_closure(a).subset(add_subgroup_set_closure(b))
} by {
    if a.subset(b) {
        add_subgroup_closure_mono(a, b)
        add_subgroup_subset_as_set_eq(add_subgroup_closure(a), add_subgroup_closure(b))
        add_subgroup_set_closure_at(a)
        add_subgroup_set_closure_at(b)
        add_subgroup_set_closure(a).subset(add_subgroup_set_closure(b))
    }
}

/// The additive subgroup set closure is unchanged after two applications.
theorem add_subgroup_set_closure_idempotent[G: AddGroup](a: Set[G]) {
    add_subgroup_set_closure(add_subgroup_set_closure(a)) = add_subgroup_set_closure(a)
} by {
    add_subgroup_set_closure_at(a)
    add_subgroup_set_closure_at(add_subgroup_set_closure(a))
    add_subgroup_closure_idempotent(a)
}

/// Additive subgroup set closure is monotone as a set map.
theorem add_subgroup_set_closure_is_monotone[G: AddGroup] {
    is_subset_monotone_map(add_subgroup_set_closure[G])
} by {
    forall(a: Set[G], b: Set[G]) {
        if a.subset(b) {
            add_subgroup_set_closure_mono(a, b)
            add_subgroup_set_closure(a).subset(add_subgroup_set_closure(b))
        }
    }
}

/// Additive subgroup set closure is extensive as a set map.
theorem add_subgroup_set_closure_is_extensive[G: AddGroup] {
    is_set_extensive_map(add_subgroup_set_closure[G])
} by {
    forall(a: Set[G]) {
        add_subgroup_set_closure_extensive(a)
        a.subset(add_subgroup_set_closure(a))
    }
}

/// Additive subgroup set closure is idempotent as a set map.
theorem add_subgroup_set_closure_is_idempotent[G: AddGroup] {
    is_set_idempotent_map(add_subgroup_set_closure[G])
} by {
    forall(a: Set[G]) {
        add_subgroup_set_closure_idempotent(a)
        add_subgroup_set_closure(add_subgroup_set_closure(a)) = add_subgroup_set_closure(a)
    }
}

/// Additive subgroup generation induces a closure operator on sets.
theorem add_subgroup_set_closure_operator[G: AddGroup] {
    is_set_closure_operator(add_subgroup_set_closure[G])
} by {
    add_subgroup_set_closure_is_monotone[G]
    add_subgroup_set_closure_is_extensive[G]
    add_subgroup_set_closure_is_idempotent[G]
}

/// The closure of the empty set is the zero additive subgroup.
theorem add_subgroup_closure_empty[G: AddGroup] {
    add_subgroup_closure(Set[G].empty_set) = zero_add_subgroup[G]
} by {
    set_subset_add_subgroup(Set[G].empty_set, zero_add_subgroup[G])
    add_subgroup_closure_subset_of_set_subset(Set[G].empty_set, zero_add_subgroup[G])
    zero_add_subgroup_subset(add_subgroup_closure(Set[G].empty_set))
    add_subgroup_subset_antisymm(add_subgroup_closure(Set[G].empty_set), zero_add_subgroup[G])
}

/// The closure of the universal set is the full additive subgroup.
theorem add_subgroup_closure_universal[G: AddGroup] {
    add_subgroup_closure(Set[G].universal_set) = full_add_subgroup[G]
} by {
    add_subgroup_subset_full(add_subgroup_closure(Set[G].universal_set))
    forall(x: G) {
        if full_add_subgroup[G].contains(x) {
            add_subgroup_closure_contains_of_set_contains(Set[G].universal_set, x)
            add_subgroup_closure(Set[G].universal_set).contains(x)
        }
    }
    add_subgroup_subset(full_add_subgroup[G], add_subgroup_closure(Set[G].universal_set))
    add_subgroup_subset_antisymm(add_subgroup_closure(Set[G].universal_set), full_add_subgroup[G])
}

/// The least additive subgroup containing two additive subgroups.
define add_subgroup_sup[G: AddGroup](a: AddSubgroup[G], b: AddSubgroup[G]) -> AddSubgroup[G] {
    add_subgroup_closure(a.as_set.union(b.as_set))
}

/// The join of two additive subgroups is the closure of the union of their underlying sets.
theorem add_subgroup_sup_eq_closure_union[G: AddGroup](a: AddSubgroup[G], b: AddSubgroup[G]) {
    add_subgroup_sup(a, b) = add_subgroup_closure(a.as_set.union(b.as_set))
}

/// The left additive subgroup is contained in the join.
theorem add_subgroup_subset_sup_left[G: AddGroup](a: AddSubgroup[G], b: AddSubgroup[G]) {
    add_subgroup_subset(a, add_subgroup_sup(a, b))
} by {
    forall(x: G) {
        if a.contains(x) {
            add_subgroup_as_set_contains_eq(a, x)
            union_contains_left(a.as_set, b.as_set, x)
            add_subgroup_closure_contains_of_set_contains(a.as_set.union(b.as_set), x)
            add_subgroup_sup(a, b).contains(x)
        }
    }
}

/// The right additive subgroup is contained in the join.
theorem add_subgroup_subset_sup_right[G: AddGroup](a: AddSubgroup[G], b: AddSubgroup[G]) {
    add_subgroup_subset(b, add_subgroup_sup(a, b))
} by {
    forall(x: G) {
        if b.contains(x) {
            add_subgroup_as_set_contains_eq(b, x)
            union_contains_right(a.as_set, b.as_set, x)
            add_subgroup_closure_contains_of_set_contains(a.as_set.union(b.as_set), x)
            add_subgroup_sup(a, b).contains(x)
        }
    }
}

/// The join is contained in every common upper bound.
theorem add_subgroup_sup_subset_of_subset_left_right[G: AddGroup](
    a: AddSubgroup[G],
    b: AddSubgroup[G],
    c: AddSubgroup[G]
) {
    add_subgroup_subset(a, c) and add_subgroup_subset(b, c) implies
        add_subgroup_subset(add_subgroup_sup(a, b), c)
} by {
    if add_subgroup_subset(a, c) and add_subgroup_subset(b, c) {
        forall(x: G) {
            if a.as_set.union(b.as_set).contains(x) {
                union_contains_eq(a.as_set, b.as_set, x)
                if a.as_set.contains(x) {
                    add_subgroup_as_set_contains_eq(a, x)
                    a.contains(x)
                    c.contains(x)
                } else {
                    b.as_set.contains(x)
                    add_subgroup_as_set_contains_eq(b, x)
                    b.contains(x)
                    c.contains(x)
                }
            }
        }
        set_subset_add_subgroup(a.as_set.union(b.as_set), c)
        add_subgroup_closure_subset_of_set_subset(a.as_set.union(b.as_set), c)
        add_subgroup_subset(add_subgroup_sup(a, b), c)
    }
}

/// Containment of a join is equivalent to containment of both additive subgroups.
theorem add_subgroup_sup_subset_iff[G: AddGroup](
    a: AddSubgroup[G],
    b: AddSubgroup[G],
    c: AddSubgroup[G]
) {
    add_subgroup_subset(add_subgroup_sup(a, b), c) =
        (add_subgroup_subset(a, c) and add_subgroup_subset(b, c))
} by {
    if add_subgroup_subset(add_subgroup_sup(a, b), c) {
        add_subgroup_subset_sup_left(a, b)
        add_subgroup_subset_trans(a, add_subgroup_sup(a, b), c)
        add_subgroup_subset_sup_right(a, b)
        add_subgroup_subset_trans(b, add_subgroup_sup(a, b), c)
        add_subgroup_subset(b, c)
        add_subgroup_subset(a, c) and add_subgroup_subset(b, c)
    }
    if add_subgroup_subset(a, c) and add_subgroup_subset(b, c) {
        add_subgroup_sup_subset_of_subset_left_right(a, b, c)
        add_subgroup_subset(add_subgroup_sup(a, b), c)
    }
}

attributes AddSubgroup[G: AddGroup] {
    /// The smallest additive subgroup containing a set.
    let closure: Set[G] -> AddSubgroup[G] = add_subgroup_closure

    /// The least additive subgroup containing this additive subgroup and another additive subgroup.
    let sup: (AddSubgroup[G], AddSubgroup[G]) -> AddSubgroup[G] = add_subgroup_sup
}

/// True if an additive subgroup is generated by a finite set.
define add_subgroup_is_finitely_generated[G: AddGroup](s: AddSubgroup[G]) -> Bool {
    exists(a: Set[G]) {
        a.is_finite and add_subgroup_closure(a) = s
    }
}

/// A finitely generated additive subgroup has a finite generating set.
theorem add_subgroup_is_finitely_generated_witness[G: AddGroup](s: AddSubgroup[G]) {
    add_subgroup_is_finitely_generated(s) implies exists(a: Set[G]) {
        a.is_finite and add_subgroup_closure(a) = s
    }
}

/// An additive subgroup equal to the closure of a finite set is finitely generated.
theorem add_subgroup_is_finitely_generated_of_closure_eq[G: AddGroup](
    a: Set[G],
    s: AddSubgroup[G]
) {
    a.is_finite and add_subgroup_closure(a) = s implies add_subgroup_is_finitely_generated(s)
} by {
    if a.is_finite and add_subgroup_closure(a) = s {
        add_subgroup_is_finitely_generated(s)
    }
}

/// Equality preserves finite generation of additive subgroups.
theorem add_subgroup_is_finitely_generated_of_eq[G: AddGroup](
    s: AddSubgroup[G],
    t: AddSubgroup[G]
) {
    add_subgroup_is_finitely_generated(s) and s = t implies add_subgroup_is_finitely_generated(t)
} by {
    if add_subgroup_is_finitely_generated(s) and s = t {
        add_subgroup_is_finitely_generated_witness(s)
        let a: Set[G] satisfy {
            a.is_finite and add_subgroup_closure(a) = s
        }
        add_subgroup_is_finitely_generated_of_closure_eq(a, t)
        add_subgroup_is_finitely_generated(t)
    }
}

/// The closure of a finite set is a finitely generated additive subgroup.
theorem add_subgroup_closure_is_finitely_generated[G: AddGroup](a: Set[G]) {
    a.is_finite implies add_subgroup_is_finitely_generated(add_subgroup_closure(a))
} by {
    if a.is_finite {
        add_subgroup_is_finitely_generated_of_closure_eq(a, add_subgroup_closure(a))
    }
}

/// The additive submonoid predicate associated to an additive subgroup.
theorem add_subgroup_add_submonoid_constraint[G: AddGroup](s: AddSubgroup[G]) {
    add_submonoid_constraint(s.contains)
} by {
    add_subgroup_contains_zero(s)
    forall(a: G, b: G) {
        if s.contains(a) and s.contains(b) {
            add_subgroup_contains_add(s, a, b)
            s.contains(a + b)
        }
    }
    add_submonoid_closure_constraint(s.contains)
}

/// The additive submonoid associated to an additive subgroup.
let add_subgroup_to_add_submonoid[G: AddGroup](s: AddSubgroup[G]) -> result: AddSubmonoid[G] satisfy {
    AddSubmonoid.new(s.contains) = Option.some(result)
} by {
    add_subgroup_add_submonoid_constraint(s)
}

/// The additive submonoid associated to an additive subgroup has the same membership predicate.
theorem add_subgroup_to_add_submonoid_contains[G: AddGroup](s: AddSubgroup[G]) {
    add_subgroup_to_add_submonoid(s).contains = s.contains
}

/// Membership in the additive submonoid associated to an additive subgroup is membership in the additive subgroup.
theorem add_subgroup_to_add_submonoid_contains_eq[G: AddGroup](s: AddSubgroup[G], x: G) {
    add_subgroup_to_add_submonoid(s).contains(x) = s.contains(x)
} by {
    add_subgroup_to_add_submonoid_contains(s)
}

/// Membership in an additive subgroup is membership in its associated additive submonoid.
theorem add_subgroup_contains_to_add_submonoid_eq[G: AddGroup](s: AddSubgroup[G], x: G) {
    s.contains(x) = add_subgroup_to_add_submonoid(s).contains(x)
} by {
    add_subgroup_to_add_submonoid_contains_eq(s, x)
}

/// The associated additive submonoid has the same underlying set as the additive subgroup.
theorem add_subgroup_to_add_submonoid_as_set[G: AddGroup](s: AddSubgroup[G]) {
    add_subgroup_to_add_submonoid(s).as_set = s.as_set
}

/// The equivalence relation on `G` defined by an additive subgroup: `a ~ b` iff `a - b` lies in the subgroup.
define add_subgroup_rel[G: AddGroup](h: AddSubgroup[G], a: G, b: G) -> Bool {
    h.contains(a - b)
}

/// The relation defined by an additive subgroup is reflexive.
theorem add_subgroup_rel_reflexive[G: AddGroup](h: AddSubgroup[G], a: G) {
    add_subgroup_rel(h, a, a)
} by {
    add_subgroup_contains_zero(h)
    h.contains(a - a)
}

/// The relation defined by an additive subgroup is symmetric.
theorem add_subgroup_rel_symmetric[G: AddGroup](h: AddSubgroup[G], a: G, b: G) {
    add_subgroup_rel(h, a, b) implies add_subgroup_rel(h, b, a)
} by {
    if add_subgroup_rel(h, a, b) {
        add_subgroup_contains_neg(h, a - b)
        h.contains(-(a - b))
        -(a - b) = b + -a
        h.contains(b - a)
    }
}

/// The relation defined by an additive subgroup is reflexive.
theorem add_subgroup_rel_is_reflexive[G: AddGroup](h: AddSubgroup[G]) {
    is_reflexive(add_subgroup_rel(h))
} by {
    forall(a: G) {
        add_subgroup_rel_reflexive(h, a)
    }
}

/// The relation defined by an additive subgroup is symmetric.
theorem add_subgroup_rel_is_symmetric[G: AddGroup](h: AddSubgroup[G]) {
    is_symmetric(add_subgroup_rel(h))
} by {
    forall(a: G, b: G) {
        if add_subgroup_rel(h, a, b) {
            add_subgroup_rel_symmetric(h, a, b)
        }
    }
}

/// Subtractions chain through an intermediate point.
theorem add_subgroup_sub_chain[G: AddGroup](a: G, b: G, c: G) {
    (a - b) + (b - c) = a - c
} by {
    (a + -b) + (b + -c) = a + (-b + b) + -c
}

/// Pointwise transitivity helper for the additive-subgroup relation.
theorem add_subgroup_rel_trans_at[G: AddGroup](h: AddSubgroup[G], a: G, b: G, c: G) {
    add_subgroup_rel(h, a, b) and add_subgroup_rel(h, b, c) implies add_subgroup_rel(h, a, c)
} by {
    if add_subgroup_rel(h, a, b) and add_subgroup_rel(h, b, c) {
        h.contains(b - c)
        add_subgroup_contains_add(h, a - b, b - c)
        h.contains((a - b) + (b - c))
        add_subgroup_sub_chain(a, b, c)
        h.contains(a - c)
    }
}

/// The relation defined by an additive subgroup is transitive.
theorem add_subgroup_rel_is_transitive[G: AddGroup](h: AddSubgroup[G]) {
    is_transitive(add_subgroup_rel(h))
} by {
    is_transitive(add_subgroup_rel(h)) = forall(x: G, y: G, z: G) {
        add_subgroup_rel(h, x, y) and add_subgroup_rel(h, y, z) implies add_subgroup_rel(h, x, z)
    }
    forall(a: G, b: G, c: G) {
        if add_subgroup_rel(h, a, b) and add_subgroup_rel(h, b, c) {
            add_subgroup_rel_trans_at(h, a, b, c)
            add_subgroup_rel(h, a, c)
        }
    }
}

/// The relation defined by an additive subgroup is an equivalence relation.
theorem add_subgroup_rel_is_equivalence[G: AddGroup](h: AddSubgroup[G]) {
    is_equivalence(add_subgroup_rel(h))
} by {
    add_subgroup_rel_is_reflexive(h)
    add_subgroup_rel_is_symmetric(h)
    add_subgroup_rel_is_transitive(h)
}

/// The quotient relation associated with an additive subgroup.
let add_subgroup_quotient_relation[G: AddGroup](h: AddSubgroup[G]) -> result: QuotientRelation[G] satisfy {
    QuotientRelation[G].new(add_subgroup_rel(h)) = Option.some(result)
} by {
    add_subgroup_rel_is_equivalence(h)
}

/// The quotient relation associated with an additive subgroup has the expected underlying relation.
theorem add_subgroup_quotient_relation_rel[G: AddGroup](h: AddSubgroup[G]) {
    add_subgroup_quotient_relation(h).rel = add_subgroup_rel(h)
} by {
    quotient_relation_new_round_trip(add_subgroup_rel(h), add_subgroup_quotient_relation(h))
}

/// In a commutative additive group, the difference of sums regroups along the operation.
theorem add_sub_distrib[G: AddCommGroup](a: G, b: G, ap: G, bp: G) {
    (a + b) - (ap + bp) = (a - ap) + (b - bp)
} by {
    (a + b) - (ap + bp) = (a + b) + -(ap + bp)
    -(ap + bp) = -ap + -bp
    (a + b) + (-ap + -bp) = a + b + -ap + -bp
    a + b + -ap = a + -ap + b
    a + -ap + b + -bp = (a - ap) + (b - bp)
}

/// The additive-subgroup quotient relation is preserved by pairwise addition (commutative case).
theorem add_subgroup_rel_add_step[G: AddCommGroup](h: AddSubgroup[G], a: G, ap: G, b: G, bp: G) {
    add_subgroup_rel(h, a, ap) and add_subgroup_rel(h, b, bp)
    implies add_subgroup_rel(h, a + b, ap + bp)
} by {
    if add_subgroup_rel(h, a, ap) and add_subgroup_rel(h, b, bp) {
        h.contains(b - bp)
        add_subgroup_contains_add(h, a - ap, b - bp)
        h.contains((a - ap) + (b - bp))
        add_sub_distrib(a, b, ap, bp)
        h.contains((a + b) - (ap + bp))
    }
}

/// The additive-subgroup quotient relation is compatible with addition (commutative case).
theorem add_subgroup_rel_respects_add[G: AddCommGroup](h: AddSubgroup[G]) {
    respects_add(add_subgroup_rel(h))
} by {
    respects_add_eq_respects_binary_op(add_subgroup_rel(h))
    forall(a: G, ap: G, b: G, bp: G) {
        if add_subgroup_rel(h, a, ap) and add_subgroup_rel(h, b, bp) {
            add_subgroup_rel_add_step(h, a, ap, b, bp)
            add_subgroup_rel(h, G.add(a, b), G.add(ap, bp))
        }
    }
    respects_binary_op(add_subgroup_rel(h), G.add)
}

/// The additive-subgroup quotient relation is an additive congruence (commutative case).
theorem add_subgroup_rel_is_add_congruence[G: AddCommGroup](h: AddSubgroup[G]) {
    is_add_congruence(add_subgroup_rel(h))
} by {
    add_subgroup_rel_is_equivalence(h)
    add_subgroup_rel_respects_add(h)
    respects_add_eq_respects_binary_op(add_subgroup_rel(h))
    is_binary_congruence(add_subgroup_rel(h), G.add)
    add_congruence_eq_binary_congruence(add_subgroup_rel(h))
}

/// In a commutative additive group, the difference of negations is the negation of the difference.
theorem add_neg_sub[G: AddCommGroup](a: G, b: G) {
    -a - -b = -(a - b)
} by {
    -(a - b) = b + -a
    -a + b = b + -a
}

/// The additive-subgroup quotient relation is preserved by negation (commutative case).
theorem add_subgroup_rel_neg_step[G: AddCommGroup](h: AddSubgroup[G], a: G, b: G) {
    add_subgroup_rel(h, a, b) implies add_subgroup_rel(h, -a, -b)
} by {
    if add_subgroup_rel(h, a, b) {
        add_subgroup_contains_neg(h, a - b)
        add_neg_sub(a, b)
        h.contains(-a - -b)
    }
}

/// The additive-subgroup quotient relation is compatible with negation (commutative case).
theorem add_subgroup_rel_respects_neg[G: AddCommGroup](h: AddSubgroup[G]) {
    respects_unary_op(add_subgroup_rel(h), G.neg)
} by {
    forall(a: G, b: G) {
        if add_subgroup_rel(h, a, b) {
            add_subgroup_rel_neg_step(h, a, b)
            add_subgroup_rel(h, G.neg(a), G.neg(b))
        }
    }
}

/// The quotient relation built from an additive subgroup respects negation as a quotient unary operation.
theorem add_subgroup_quotient_relation_respects_neg[G: AddCommGroup](h: AddSubgroup[G]) {
    quotient_unary_respects(add_subgroup_quotient_relation(h), G.neg)
} by {
    quotient_unary_respects_eq_respects_unary_op(add_subgroup_quotient_relation(h), G.neg)
    add_subgroup_rel_respects_neg(h)
    add_subgroup_quotient_relation_rel(h)
}

/// The quotient relation built from an additive subgroup respects addition as a quotient binary operation.
theorem add_subgroup_quotient_relation_respects_add[G: AddCommGroup](h: AddSubgroup[G]) {
    quotient_binary_respects(add_subgroup_quotient_relation(h), G.add)
} by {
    quotient_binary_respects_eq_respects_binary_op(add_subgroup_quotient_relation(h), G.add)
    add_subgroup_rel_respects_add(h)
    respects_add_eq_respects_binary_op(add_subgroup_rel(h))
    add_subgroup_quotient_relation_rel(h)
}

/// Addition on the relation-indexed quotient by an additive subgroup.
define add_subgroup_quotient_add[G: AddCommGroup](h: AddSubgroup[G]) ->
    (QuotientOver[G], QuotientOver[G]) -> QuotientOver[G] {
    quotient_over_binary_op(G.add)
}

/// Addition in the additive-subgroup quotient agrees with addition of representatives.
theorem add_subgroup_quotient_add_projection[G: AddCommGroup](h: AddSubgroup[G], a: G, b: G) {
    add_subgroup_quotient_add(h,
        quotient_over_mk(add_subgroup_quotient_relation(h), a),
        quotient_over_mk(add_subgroup_quotient_relation(h), b)
    ) = quotient_over_mk(add_subgroup_quotient_relation(h), a + b)
} by {
    add_subgroup_quotient_relation_respects_add(h)
    quotient_over_binary_op_projection(add_subgroup_quotient_relation(h), G.add, a, b)
    G.add(a, b) = a + b
}

/// Addition in the additive-subgroup quotient respects related left representatives.
theorem add_subgroup_quotient_add_projection_left[G: AddCommGroup](h: AddSubgroup[G], a1: G, a2: G, b: G) {
    add_subgroup_rel(h, a1, a2) implies
    add_subgroup_quotient_add(h,
        quotient_over_mk(add_subgroup_quotient_relation(h), a1),
        quotient_over_mk(add_subgroup_quotient_relation(h), b)
    ) = add_subgroup_quotient_add(h,
        quotient_over_mk(add_subgroup_quotient_relation(h), a2),
        quotient_over_mk(add_subgroup_quotient_relation(h), b)
    )
} by {
    if add_subgroup_rel(h, a1, a2) {
        add_subgroup_quotient_relation_rel(h)
        add_subgroup_quotient_relation(h).rel(a1, a2)
        add_subgroup_quotient_relation_respects_add(h)
        quotient_over_binary_op_projection_left(add_subgroup_quotient_relation(h), G.add, a1, a2, b)
    }
}

/// Addition in the additive-subgroup quotient respects related right representatives.
theorem add_subgroup_quotient_add_projection_right[G: AddCommGroup](h: AddSubgroup[G], a: G, b1: G, b2: G) {
    add_subgroup_rel(h, b1, b2) implies
    add_subgroup_quotient_add(h,
        quotient_over_mk(add_subgroup_quotient_relation(h), a),
        quotient_over_mk(add_subgroup_quotient_relation(h), b1)
    ) = add_subgroup_quotient_add(h,
        quotient_over_mk(add_subgroup_quotient_relation(h), a),
        quotient_over_mk(add_subgroup_quotient_relation(h), b2)
    )
} by {
    if add_subgroup_rel(h, b1, b2) {
        add_subgroup_quotient_relation_rel(h)
        add_subgroup_quotient_relation(h).rel(b1, b2)
        add_subgroup_quotient_relation_respects_add(h)
        quotient_over_binary_op_projection_right(add_subgroup_quotient_relation(h), G.add, a, b1, b2)
    }
}

/// Negation on the relation-indexed quotient by an additive subgroup.
define add_subgroup_quotient_neg[G: AddCommGroup](h: AddSubgroup[G]) ->
    QuotientOver[G] -> QuotientOver[G] {
    quotient_over_unary_op(add_subgroup_quotient_relation(h), G.neg)
}

/// Negation in the additive-subgroup quotient agrees with negation of representatives.
theorem add_subgroup_quotient_neg_projection[G: AddCommGroup](h: AddSubgroup[G], a: G) {
    add_subgroup_quotient_neg(h, quotient_over_mk(add_subgroup_quotient_relation(h), a)) =
        quotient_over_mk(add_subgroup_quotient_relation(h), -a)
} by {
    add_subgroup_quotient_relation_respects_neg(h)
    quotient_over_unary_op_projection(add_subgroup_quotient_relation(h), G.neg, a)
    G.neg(a) = -a
}

/// Negation in the additive-subgroup quotient respects related representatives.
theorem add_subgroup_quotient_neg_projection_compatible[G: AddCommGroup](h: AddSubgroup[G], a: G, b: G) {
    add_subgroup_rel(h, a, b) implies
    add_subgroup_quotient_neg(h, quotient_over_mk(add_subgroup_quotient_relation(h), a)) =
        add_subgroup_quotient_neg(h, quotient_over_mk(add_subgroup_quotient_relation(h), b))
} by {
    if add_subgroup_rel(h, a, b) {
        add_subgroup_quotient_relation_rel(h)
        add_subgroup_quotient_relation(h).rel(a, b)
        add_subgroup_quotient_relation_respects_neg(h)
        quotient_over_unary_op_projection_compatible(add_subgroup_quotient_relation(h), G.neg, a, b)
    }
}

/// The zero element of the additive-subgroup quotient.
define add_subgroup_quotient_zero[G: AddCommGroup](h: AddSubgroup[G]) -> QuotientOver[G] {
    quotient_over_mk(add_subgroup_quotient_relation(h), G.0)
}

/// Addition on the additive-subgroup quotient is associative.
theorem add_subgroup_quotient_add_assoc[G: AddCommGroup](h: AddSubgroup[G], a: G, b: G, c: G) {
    add_subgroup_quotient_add(h,
        add_subgroup_quotient_add(h,
            quotient_over_mk(add_subgroup_quotient_relation(h), a),
            quotient_over_mk(add_subgroup_quotient_relation(h), b)),
        quotient_over_mk(add_subgroup_quotient_relation(h), c)) =
    add_subgroup_quotient_add(h,
        quotient_over_mk(add_subgroup_quotient_relation(h), a),
        add_subgroup_quotient_add(h,
            quotient_over_mk(add_subgroup_quotient_relation(h), b),
            quotient_over_mk(add_subgroup_quotient_relation(h), c)))
} by {
    add_subgroup_quotient_add_projection(h, a, b)
    add_subgroup_quotient_add_projection(h, a + b, c)
    add_subgroup_quotient_add_projection(h, b, c)
    add_subgroup_quotient_add_projection(h, a, b + c)
    quotient_over_mk(add_subgroup_quotient_relation(h), (a + b) + c) =
        quotient_over_mk(add_subgroup_quotient_relation(h), a + (b + c))
}

/// The quotient zero is a left identity for quotient addition.
theorem add_subgroup_quotient_zero_add[G: AddCommGroup](h: AddSubgroup[G], a: G) {
    add_subgroup_quotient_add(h,
        quotient_over_mk(add_subgroup_quotient_relation(h), G.0),
        quotient_over_mk(add_subgroup_quotient_relation(h), a)) =
        quotient_over_mk(add_subgroup_quotient_relation(h), a)
} by {
    add_subgroup_quotient_add_projection(h, G.0, a)
}

/// The quotient zero is a right identity for quotient addition.
theorem add_subgroup_quotient_add_zero[G: AddCommGroup](h: AddSubgroup[G], a: G) {
    add_subgroup_quotient_add(h,
        quotient_over_mk(add_subgroup_quotient_relation(h), a),
        quotient_over_mk(add_subgroup_quotient_relation(h), G.0)) =
        quotient_over_mk(add_subgroup_quotient_relation(h), a)
} by {
    add_subgroup_quotient_add_projection(h, a, G.0)
}

/// The quotient negation is a left inverse for quotient addition.
theorem add_subgroup_quotient_neg_add[G: AddCommGroup](h: AddSubgroup[G], a: G) {
    add_subgroup_quotient_add(h,
        add_subgroup_quotient_neg(h, quotient_over_mk(add_subgroup_quotient_relation(h), a)),
        quotient_over_mk(add_subgroup_quotient_relation(h), a)) =
        quotient_over_mk(add_subgroup_quotient_relation(h), G.0)
} by {
    add_subgroup_quotient_neg_projection(h, a)
    add_subgroup_quotient_add_projection(h, -a, a)
}

/// The quotient negation is a right inverse for quotient addition.
theorem add_subgroup_quotient_add_neg[G: AddCommGroup](h: AddSubgroup[G], a: G) {
    add_subgroup_quotient_add(h,
        quotient_over_mk(add_subgroup_quotient_relation(h), a),
        add_subgroup_quotient_neg(h, quotient_over_mk(add_subgroup_quotient_relation(h), a))) =
        quotient_over_mk(add_subgroup_quotient_relation(h), G.0)
} by {
    add_subgroup_quotient_neg_projection(h, a)
    add_subgroup_quotient_add_projection(h, a, -a)
}

/// Addition on the additive-subgroup quotient is commutative.
theorem add_subgroup_quotient_add_comm[G: AddCommGroup](h: AddSubgroup[G], a: G, b: G) {
    add_subgroup_quotient_add(h,
        quotient_over_mk(add_subgroup_quotient_relation(h), a),
        quotient_over_mk(add_subgroup_quotient_relation(h), b)) =
    add_subgroup_quotient_add(h,
        quotient_over_mk(add_subgroup_quotient_relation(h), b),
        quotient_over_mk(add_subgroup_quotient_relation(h), a))
} by {
    add_subgroup_quotient_add_projection(h, a, b)
    add_subgroup_quotient_add_projection(h, b, a)
}

/// The inverse image membership predicate of an additive subgroup under an additive group homomorphism.
define add_subgroup_preimage_contains[A: AddGroup, B: AddGroup](
    f: AddGroupHom[A, B],
    s: AddSubgroup[B],
    a: A
) -> Bool {
    s.contains(f.hom(a))
}

/// The inverse image of an additive subgroup contains zero.
theorem add_subgroup_preimage_zero_constraint[A: AddGroup, B: AddGroup](
    f: AddGroupHom[A, B],
    s: AddSubgroup[B]
) {
    add_zero_constraint(add_subgroup_preimage_contains(f, s))
} by {
    add_group_hom_zero(f)
    add_subgroup_contains_zero(s)
}

/// The inverse image of an additive subgroup is closed under addition.
theorem add_subgroup_preimage_closure_constraint[A: AddGroup, B: AddGroup](
    f: AddGroupHom[A, B],
    s: AddSubgroup[B]
) {
    add_closure_constraint(add_subgroup_preimage_contains(f, s))
} by {
    forall(a: A, b: A) {
        if add_subgroup_preimage_contains(f, s, a) and add_subgroup_preimage_contains(f, s, b) {
            s.contains(f.hom(b))
            add_subgroup_contains_add(s, f.hom(a), f.hom(b))
            s.contains(f.hom(a) + f.hom(b))
            add_group_hom_add(f, a, b)
            add_subgroup_preimage_contains(f, s, a + b)
        }
    }
}

/// The inverse image of an additive subgroup is closed under negation.
theorem add_subgroup_preimage_neg_constraint[A: AddGroup, B: AddGroup](
    f: AddGroupHom[A, B],
    s: AddSubgroup[B]
) {
    neg_constraint(add_subgroup_preimage_contains(f, s))
} by {
    forall(a: A) {
        if add_subgroup_preimage_contains(f, s, a) {
            add_subgroup_contains_neg(s, f.hom(a))
            add_group_hom_neg(f, a)
            add_subgroup_preimage_contains(f, s, -a)
        }
    }
}

/// The inverse image of an additive subgroup is an additive subgroup.
theorem add_subgroup_preimage_constraint[A: AddGroup, B: AddGroup](
    f: AddGroupHom[A, B],
    s: AddSubgroup[B]
) {
    add_subgroup_constraint(add_subgroup_preimage_contains(f, s))
} by {
    add_subgroup_preimage_zero_constraint(f, s)
    add_subgroup_preimage_closure_constraint(f, s)
    add_subgroup_preimage_neg_constraint(f, s)
}

/// The inverse image of an additive subgroup under an additive group homomorphism.
let add_subgroup_preimage[A: AddGroup, B: AddGroup](f: AddGroupHom[A, B], s: AddSubgroup[B]) -> result: AddSubgroup[A] satisfy {
    AddSubgroup.new(add_subgroup_preimage_contains(f, s)) = Option.some(result)
} by {
    add_subgroup_preimage_constraint(f, s)
}

/// Membership in an inverse image means the mapped element belongs to the target additive subgroup.
theorem add_subgroup_preimage_contains_eq[A: AddGroup, B: AddGroup](
    f: AddGroupHom[A, B],
    s: AddSubgroup[B],
    a: A
) {
    add_subgroup_preimage(f, s).contains(a) = s.contains(f.hom(a))
} by {
    add_subgroup_preimage(f, s).contains(a) = add_subgroup_preimage_contains(f, s, a)
}

/// Membership in a preimage follows from membership of the mapped element.
theorem add_subgroup_preimage_contains_of_contains_map[A: AddGroup, B: AddGroup](
    f: AddGroupHom[A, B],
    s: AddSubgroup[B],
    a: A
) {
    s.contains(f.hom(a)) implies add_subgroup_preimage(f, s).contains(a)
} by {
    if s.contains(f.hom(a)) {
        add_subgroup_preimage_contains_eq(f, s, a)
        add_subgroup_preimage(f, s).contains(a)
    }
}

/// Membership in the target follows from membership in the preimage.
theorem add_subgroup_contains_map_of_preimage_contains[A: AddGroup, B: AddGroup](
    f: AddGroupHom[A, B],
    s: AddSubgroup[B],
    a: A
) {
    add_subgroup_preimage(f, s).contains(a) implies s.contains(f.hom(a))
} by {
    if add_subgroup_preimage(f, s).contains(a) {
        add_subgroup_preimage_contains_eq(f, s, a)
        s.contains(f.hom(a))
    }
}

/// Inverse images are monotone with respect to additive subgroup containment.
theorem add_subgroup_preimage_subset_of_subset[A: AddGroup, B: AddGroup](
    f: AddGroupHom[A, B],
    s: AddSubgroup[B],
    t: AddSubgroup[B]
) {
    add_subgroup_subset(s, t) implies
        add_subgroup_subset(add_subgroup_preimage(f, s), add_subgroup_preimage(f, t))
} by {
    if add_subgroup_subset(s, t) {
        forall(a: A) {
            if add_subgroup_preimage(f, s).contains(a) {
                add_subgroup_preimage_contains_eq(f, s, a)
                add_subgroup_subset(s, t) = forall(x: B) {
                    s.contains(x) implies t.contains(x)
                }
                add_subgroup_preimage_contains_eq(f, t, a)
                add_subgroup_preimage(f, t).contains(a)
            }
        }
    }
}

/// The inverse image of an intersection is the intersection of the inverse images.
theorem add_subgroup_preimage_intersection[A: AddGroup, B: AddGroup](
    f: AddGroupHom[A, B],
    s: AddSubgroup[B],
    t: AddSubgroup[B]
) {
    add_subgroup_preimage(f, s.intersection(t)) =
        add_subgroup_preimage(f, s).intersection(add_subgroup_preimage(f, t))
} by {
    forall(a: A) {
        add_subgroup_preimage_contains_eq(f, s.intersection(t), a)
        add_subgroup_preimage_contains_eq(f, s, a)
        add_subgroup_preimage_contains_eq(f, t, a)
        add_subgroup_intersection_contains_eq(s, t, f.hom(a))
        add_subgroup_intersection_contains_eq(add_subgroup_preimage(f, s), add_subgroup_preimage(f, t), a)
        add_subgroup_preimage(f, s.intersection(t)).contains(a) =
            add_subgroup_preimage(f, s).intersection(add_subgroup_preimage(f, t)).contains(a)
    }
    add_subgroup_ext(add_subgroup_preimage(f, s.intersection(t)),
        add_subgroup_preimage(f, s).intersection(add_subgroup_preimage(f, t)))
}

/// The inverse image of the full additive subgroup is the full additive subgroup.
theorem add_subgroup_preimage_full[A: AddGroup, B: AddGroup](f: AddGroupHom[A, B]) {
    add_subgroup_preimage(f, full_add_subgroup[B]) = full_add_subgroup[A]
} by {
    add_subgroup_subset_full(add_subgroup_preimage(f, full_add_subgroup[B]))
    forall(a: A) {
        if full_add_subgroup[A].contains(a) {
            full_add_subgroup_contains_everything(f.hom(a))
            add_subgroup_preimage_contains_eq(f, full_add_subgroup[B], a)
            add_subgroup_preimage(f, full_add_subgroup[B]).contains(a)
        }
    }
    add_subgroup_subset(full_add_subgroup[A], add_subgroup_preimage(f, full_add_subgroup[B]))
    add_subgroup_subset_antisymm(add_subgroup_preimage(f, full_add_subgroup[B]), full_add_subgroup[A])
}

/// The forward image membership predicate of an additive subgroup under an
/// additive group homomorphism.
define add_subgroup_image_contains[A: AddGroup, B: AddGroup](
    f: AddGroupHom[A, B],
    s: AddSubgroup[A],
    y: B
) -> Bool {
    exists(x: A) {
        s.contains(x) and y = f.hom(x)
    }
}

/// The image of an additive subgroup contains zero.
theorem add_subgroup_image_contains_zero[A: AddGroup, B: AddGroup](
    f: AddGroupHom[A, B],
    s: AddSubgroup[A]
) {
    add_subgroup_image_contains(f, s, B.0)
} by {
    add_subgroup_contains_zero(s)
    add_group_hom_zero(f)
}

/// The image of two source-subgroup elements is closed under addition.
theorem add_subgroup_image_contains_add[A: AddGroup, B: AddGroup](
    f: AddGroupHom[A, B],
    s: AddSubgroup[A],
    a: B,
    b: B
) {
    add_subgroup_image_contains(f, s, a) and add_subgroup_image_contains(f, s, b)
        implies add_subgroup_image_contains(f, s, a + b)
} by {
    if add_subgroup_image_contains(f, s, a) and add_subgroup_image_contains(f, s, b) {
        let xa: A satisfy { s.contains(xa) and a = f.hom(xa) }
        let xb: A satisfy { s.contains(xb) and b = f.hom(xb) }
        add_subgroup_contains_add(s, xa, xb)
        add_group_hom_add(f, xa, xb)
        exists(x: A) {
            s.contains(x) and a + b = f.hom(x)
        }
        add_subgroup_image_contains(f, s, a + b)
    }
}

/// The image of a source-subgroup element is closed under negation.
theorem add_subgroup_image_contains_neg[A: AddGroup, B: AddGroup](
    f: AddGroupHom[A, B],
    s: AddSubgroup[A],
    a: B
) {
    add_subgroup_image_contains(f, s, a) implies add_subgroup_image_contains(f, s, -a)
} by {
    if add_subgroup_image_contains(f, s, a) {
        let x: A satisfy { s.contains(x) and a = f.hom(x) }
        add_subgroup_contains_neg(s, x)
        add_group_hom_neg(f, x)
        exists(w: A) {
            s.contains(w) and -a = f.hom(w)
        }
        add_subgroup_image_contains(f, s, -a)
    }
}

/// The image of an additive subgroup contains zero.
theorem add_subgroup_image_zero_constraint[A: AddGroup, B: AddGroup](
    f: AddGroupHom[A, B],
    s: AddSubgroup[A]
) {
    add_zero_constraint(add_subgroup_image_contains(f, s))
} by {
    add_subgroup_image_contains_zero(f, s)
}

/// The image of an additive subgroup is closed under addition.
theorem add_subgroup_image_closure_constraint[A: AddGroup, B: AddGroup](
    f: AddGroupHom[A, B],
    s: AddSubgroup[A]
) {
    add_closure_constraint(add_subgroup_image_contains(f, s))
} by {
    let p: B -> Bool = add_subgroup_image_contains(f, s)
    forall(a: B, b: B) {
        if p(a) and p(b) {
            p(b) = add_subgroup_image_contains(f, s, b)
            add_subgroup_image_contains_add(f, s, a, b)
            p(a + b)
        }
        p(a) and p(b) implies p(a + b)
    }
    add_closure_constraint(p)
}

/// The image of an additive subgroup is closed under negation.
theorem add_subgroup_image_neg_constraint[A: AddGroup, B: AddGroup](
    f: AddGroupHom[A, B],
    s: AddSubgroup[A]
) {
    neg_constraint(add_subgroup_image_contains(f, s))
} by {
    let p: B -> Bool = add_subgroup_image_contains(f, s)
    forall(a: B) {
        if p(a) {
            add_subgroup_image_contains_neg(f, s, a)
            p(-a)
        }
    }
}

/// The forward image predicate of an additive subgroup is an additive subgroup predicate.
theorem add_subgroup_image_constraint[A: AddGroup, B: AddGroup](
    f: AddGroupHom[A, B],
    s: AddSubgroup[A]
) {
    add_subgroup_constraint(add_subgroup_image_contains(f, s))
} by {
    add_subgroup_image_zero_constraint(f, s)
    add_subgroup_image_closure_constraint(f, s)
    add_subgroup_image_neg_constraint(f, s)
}

/// The forward image of an additive subgroup under an additive group homomorphism.
let add_subgroup_image[A: AddGroup, B: AddGroup](f: AddGroupHom[A, B], s: AddSubgroup[A]) -> result: AddSubgroup[B] satisfy {
    AddSubgroup.new(add_subgroup_image_contains(f, s)) = Option.some(result)
} by {
    add_subgroup_image_constraint(f, s)
}

/// Image-subgroup membership is exactly the image predicate.
theorem add_subgroup_image_contains_eq[A: AddGroup, B: AddGroup](
    f: AddGroupHom[A, B],
    s: AddSubgroup[A],
    y: B
) {
    add_subgroup_image(f, s).contains(y) = exists(x: A) {
        s.contains(x) and y = f.hom(x)
    }
} by {
    Option.some(add_subgroup_image(f, s)) = AddSubgroup.new(add_subgroup_image_contains(f, s))
    add_subgroup_image(f, s).contains(y) = add_subgroup_image_contains(f, s, y)
}

/// The image of an element in the source subgroup lies in the image subgroup.
theorem add_subgroup_image_contains_map[A: AddGroup, B: AddGroup](
    f: AddGroupHom[A, B],
    s: AddSubgroup[A],
    x: A
) {
    s.contains(x) implies add_subgroup_image(f, s).contains(f.hom(x))
} by {
    if s.contains(x) {
        add_subgroup_image_contains_eq(f, s, f.hom(x))
        add_subgroup_image(f, s).contains(f.hom(x))
    }
}

/// If the image subgroup is contained in a target subgroup, the source subgroup
/// is contained in the target preimage.
theorem add_subgroup_subset_preimage_of_image_subset[A: AddGroup, B: AddGroup](
    f: AddGroupHom[A, B],
    s: AddSubgroup[A],
    t: AddSubgroup[B]
) {
    add_subgroup_subset(add_subgroup_image(f, s), t) implies
        add_subgroup_subset(s, add_subgroup_preimage(f, t))
} by {
    if add_subgroup_subset(add_subgroup_image(f, s), t) {
        forall(x: A) {
            if s.contains(x) {
                add_subgroup_image_contains_map(f, s, x)
                (forall(y: B) { not add_subgroup_image(f, s).contains(y) or t.contains(y) }) = true
                add_subgroup_preimage_contains_eq(f, t, x)
                add_subgroup_preimage(f, t).contains(x)
            }
        }
    }
}

/// If a source subgroup is contained in the preimage of a target subgroup, its
/// image subgroup is contained in the target subgroup.
theorem add_subgroup_image_subset_of_subset_preimage[A: AddGroup, B: AddGroup](
    f: AddGroupHom[A, B],
    s: AddSubgroup[A],
    t: AddSubgroup[B]
) {
    add_subgroup_subset(s, add_subgroup_preimage(f, t)) implies
        add_subgroup_subset(add_subgroup_image(f, s), t)
} by {
    if add_subgroup_subset(s, add_subgroup_preimage(f, t)) {
        forall(y: B) {
            if add_subgroup_image(f, s).contains(y) {
                add_subgroup_image_contains_eq(f, s, y)
                let x: A satisfy { s.contains(x) and y = f.hom(x) }
                (forall(a: A) { not s.contains(a) or add_subgroup_preimage(f, t).contains(a) }) = true
                add_subgroup_preimage_contains_eq(f, t, x)
                t.contains(f.hom(x))
                t.contains(y)
            }
        }
    }
}

/// Image and preimage under an additive group homomorphism form a Galois connection.
theorem add_subgroup_image_subset_iff_subset_preimage[A: AddGroup, B: AddGroup](
    f: AddGroupHom[A, B],
    s: AddSubgroup[A],
    t: AddSubgroup[B]
) {
    add_subgroup_subset(add_subgroup_image(f, s), t) =
        add_subgroup_subset(s, add_subgroup_preimage(f, t))
} by {
    if add_subgroup_subset(add_subgroup_image(f, s), t) {
        add_subgroup_subset_preimage_of_image_subset(f, s, t)
        add_subgroup_subset(s, add_subgroup_preimage(f, t))
    }
    if add_subgroup_subset(s, add_subgroup_preimage(f, t)) {
        add_subgroup_image_subset_of_subset_preimage(f, s, t)
        add_subgroup_subset(add_subgroup_image(f, s), t)
    }
}

/// Forward images are monotone with respect to additive subgroup containment.
theorem add_subgroup_image_subset_of_subset[A: AddGroup, B: AddGroup](
    f: AddGroupHom[A, B],
    s: AddSubgroup[A],
    t: AddSubgroup[A]
) {
    add_subgroup_subset(s, t) implies
        add_subgroup_subset(add_subgroup_image(f, s), add_subgroup_image(f, t))
} by {
    if add_subgroup_subset(s, t) {
        forall(y: B) {
            if add_subgroup_image(f, s).contains(y) {
                add_subgroup_image_contains_eq(f, s, y)
                let x: A satisfy { s.contains(x) and y = f.hom(x) }
                (forall(a: A) { not s.contains(a) or t.contains(a) }) = true
                add_subgroup_image_contains_map(f, t, x)
                add_subgroup_image(f, t).contains(f.hom(x))
                add_subgroup_image(f, t).contains(y)
            }
        }
    }
}

/// The kernel of an additive group homomorphism as an additive subgroup.
define add_group_hom_kernel[A: AddGroup, B: AddGroup](f: AddGroupHom[A, B]) -> AddSubgroup[A] {
    add_subgroup_preimage(f, zero_add_subgroup[B])
}

/// Membership in the kernel means the homomorphism maps the element to zero.
theorem add_group_hom_kernel_contains_eq[A: AddGroup, B: AddGroup](f: AddGroupHom[A, B], a: A) {
    add_group_hom_kernel(f).contains(a) = (f.hom(a) = B.0)
} by {
    add_subgroup_preimage_contains_eq(f, zero_add_subgroup[B], a)
    if add_group_hom_kernel(f).contains(a) {
        zero_add_subgroup[B].contains(f.hom(a))
        zero_add_subgroup_only_has_zero(f.hom(a))
        f.hom(a) = B.0
    }
    if f.hom(a) = B.0 {
        add_subgroup_contains_zero(zero_add_subgroup[B])
        add_group_hom_kernel(f).contains(a)
    }
}

/// An element belongs to the kernel when its image is zero.
theorem add_group_hom_kernel_contains_of_maps_to_zero[A: AddGroup, B: AddGroup](f: AddGroupHom[A, B], a: A) {
    f.hom(a) = B.0 implies add_group_hom_kernel(f).contains(a)
} by {
    if f.hom(a) = B.0 {
        add_group_hom_kernel_contains_eq(f, a)
        add_group_hom_kernel(f).contains(a)
    }
}

/// A kernel element maps to zero.
theorem add_group_hom_maps_to_zero_of_kernel_contains[A: AddGroup, B: AddGroup](f: AddGroupHom[A, B], a: A) {
    add_group_hom_kernel(f).contains(a) implies f.hom(a) = B.0
} by {
    if add_group_hom_kernel(f).contains(a) {
        add_group_hom_kernel_contains_eq(f, a)
        f.hom(a) = B.0
    }
}


/// Additive subgroup join is commutative.
theorem add_subgroup_sup_comm[G: AddGroup](a: AddSubgroup[G], b: AddSubgroup[G]) {
    add_subgroup_sup(a, b) = add_subgroup_sup(b, a)
} by {
    add_subgroup_subset_sup_right(b, a)
    add_subgroup_subset_sup_left(b, a)
    add_subgroup_sup_subset_of_subset_left_right(a, b, add_subgroup_sup(b, a))
    add_subgroup_subset_sup_right(a, b)
    add_subgroup_subset_sup_left(a, b)
    add_subgroup_sup_subset_of_subset_left_right(b, a, add_subgroup_sup(a, b))
    add_subgroup_subset_antisymm(add_subgroup_sup(a, b), add_subgroup_sup(b, a))
}

/// Joining an additive subgroup with itself gives the same subgroup.
theorem add_subgroup_sup_idempotent[G: AddGroup](a: AddSubgroup[G]) {
    add_subgroup_sup(a, a) = a
} by {
    add_subgroup_subset_refl(a)
    add_subgroup_sup_subset_of_subset_left_right(a, a, a)
    add_subgroup_subset_sup_left(a, a)
    add_subgroup_subset_antisymm(add_subgroup_sup(a, a), a)
}

/// Additive subgroup join is associative.
theorem add_subgroup_sup_assoc[G: AddGroup](
    a: AddSubgroup[G],
    b: AddSubgroup[G],
    c: AddSubgroup[G]
) {
    add_subgroup_sup(add_subgroup_sup(a, b), c) = add_subgroup_sup(a, add_subgroup_sup(b, c))
} by {
    add_subgroup_subset_sup_left(a, add_subgroup_sup(b, c))
    add_subgroup_subset_sup_right(a, add_subgroup_sup(b, c))
    add_subgroup_subset_sup_left(b, c)
    add_subgroup_subset_sup_right(b, c)
    add_subgroup_subset_trans(b, add_subgroup_sup(b, c), add_subgroup_sup(a, add_subgroup_sup(b, c)))
    add_subgroup_subset_trans(c, add_subgroup_sup(b, c), add_subgroup_sup(a, add_subgroup_sup(b, c)))
    add_subgroup_sup_subset_of_subset_left_right(a, b, add_subgroup_sup(a, add_subgroup_sup(b, c)))
    add_subgroup_sup_subset_of_subset_left_right(add_subgroup_sup(a, b), c, add_subgroup_sup(a, add_subgroup_sup(b, c)))
    add_subgroup_subset_sup_left(add_subgroup_sup(a, b), c)
    add_subgroup_subset_sup_right(add_subgroup_sup(a, b), c)
    add_subgroup_subset_sup_left(a, b)
    add_subgroup_subset_sup_right(a, b)
    add_subgroup_subset_trans(a, add_subgroup_sup(a, b), add_subgroup_sup(add_subgroup_sup(a, b), c))
    add_subgroup_subset_trans(b, add_subgroup_sup(a, b), add_subgroup_sup(add_subgroup_sup(a, b), c))
    add_subgroup_sup_subset_of_subset_left_right(b, c, add_subgroup_sup(add_subgroup_sup(a, b), c))
    add_subgroup_sup_subset_of_subset_left_right(a, add_subgroup_sup(b, c), add_subgroup_sup(add_subgroup_sup(a, b), c))
    add_subgroup_subset_antisymm(add_subgroup_sup(add_subgroup_sup(a, b), c), add_subgroup_sup(a, add_subgroup_sup(b, c)))
}
