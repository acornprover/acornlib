from algebra.mul import Mul
from algebra.one import One
from algebra.zero import Zero
from algebra.add import Add
from algebra.semigroup import Semigroup
from algebra.comm_semigroup import CommSemigroup
from algebra.monoid.monoid import Monoid
from algebra.group import Group, inverse_left, inverse_mul
from semiring import Semiring
from data.basic.relation_transport import preserves_binary_op

/// Multiplication with the order of factors reversed.
define opposite_mul[M: Mul](a: M, b: M) -> M {
    b * a
}

/// Opposite multiplication is the reversed original multiplication.
theorem opposite_mul_eq[M: Mul](a: M, b: M) {
    opposite_mul(a, b) = b * a
}

/// Opposite multiplication with reversed arguments is the original multiplication.
theorem opposite_mul_rev_eq[M: Mul](a: M, b: M) {
    opposite_mul(b, a) = a * b
}

/// Opposite multiplication is associative when the original multiplication is associative.
theorem opposite_mul_assoc[M: Semigroup](a: M, b: M, c: M) {
    opposite_mul(a, opposite_mul(b, c)) = opposite_mul(opposite_mul(a, b), c)
} by {
    opposite_mul(a, opposite_mul(b, c)) = opposite_mul(b, c) * a
    opposite_mul(b, c) = c * b
    opposite_mul(a, opposite_mul(b, c)) = (c * b) * a
    opposite_mul(opposite_mul(a, b), c) = c * opposite_mul(a, b)
    opposite_mul(a, b) = b * a
    opposite_mul(opposite_mul(a, b), c) = c * (b * a)
    (c * b) * a = c * (b * a)
}

/// Opposite multiplication is commutative when the original multiplication is commutative.
theorem opposite_mul_comm[M: CommSemigroup](a: M, b: M) {
    opposite_mul(a, b) = opposite_mul(b, a)
} by {
    opposite_mul(a, b) = b * a
    opposite_mul(b, a) = a * b
    b * a = a * b
}

/// Opposite multiplication agrees with the original multiplication in a commutative semigroup.
theorem opposite_mul_eq_mul[M: CommSemigroup](a: M, b: M) {
    opposite_mul(a, b) = a * b
} by {
    opposite_mul(a, b) = b * a
    b * a = a * b
}

/// The identity is a right identity for opposite multiplication.
theorem opposite_mul_one_right[M: Monoid](a: M) {
    opposite_mul(a, M.1) = a
} by {
    opposite_mul(a, M.1) = M.1 * a
    M.1 * a = a
}

/// The identity is a left identity for opposite multiplication.
theorem opposite_mul_one_left[M: Monoid](a: M) {
    opposite_mul(M.1, a) = a
} by {
    opposite_mul(M.1, a) = a * M.1
    a * M.1 = a
}

/// The inverse is a right inverse for opposite multiplication.
theorem opposite_mul_inverse_right[G: Group](a: G) {
    opposite_mul(a, a.inverse) = G.1
} by {
    opposite_mul(a, a.inverse) = a.inverse * a
    inverse_left(a)
    a.inverse * a = G.1
}

/// The inverse is a left inverse for opposite multiplication.
theorem opposite_mul_inverse_left[G: Group](a: G) {
    opposite_mul(a.inverse, a) = G.1
} by {
    opposite_mul(a.inverse, a) = a * a.inverse
    a * a.inverse = G.1
}

/// Inversion reverses opposite multiplication.
theorem inverse_opposite_mul[G: Group](a: G, b: G) {
    opposite_mul(a, b).inverse = opposite_mul(b.inverse, a.inverse)
} by {
    opposite_mul(a, b) = b * a
    opposite_mul(a, b).inverse = (b * a).inverse
    inverse_mul(b, a)
    (b * a).inverse = a.inverse * b.inverse
    opposite_mul(b.inverse, a.inverse) = a.inverse * b.inverse
}

/// Inversion of original multiplication is opposite multiplication of inverses.
theorem inverse_mul_eq_opposite_inverse_mul[G: Group](a: G, b: G) {
    (a * b).inverse = opposite_mul(a.inverse, b.inverse)
} by {
    inverse_mul(a, b)
    (a * b).inverse = b.inverse * a.inverse
    opposite_mul(a.inverse, b.inverse) = b.inverse * a.inverse
}

/// Opposite multiplication by zero on the right is zero.
theorem opposite_mul_zero_right[R: Semiring](a: R) {
    opposite_mul(a, R.0) = R.0
} by {
    opposite_mul(a, R.0) = R.0 * a
    R.0 * a = R.0
}

/// Opposite multiplication by zero on the left is zero.
theorem opposite_mul_zero_left[R: Semiring](a: R) {
    opposite_mul(R.0, a) = R.0
} by {
    opposite_mul(R.0, a) = a * R.0
    a * R.0 = R.0
}

/// Opposite multiplication distributes over addition from the left.
theorem opposite_mul_add_distrib_left[R: Semiring](a: R, b: R, c: R) {
    opposite_mul(a, b + c) = opposite_mul(a, b) + opposite_mul(a, c)
} by {
    opposite_mul(a, b + c) = (b + c) * a
    (b + c) * a = b * a + c * a
    opposite_mul(a, b) = b * a
    opposite_mul(a, c) = c * a
}

/// Opposite multiplication distributes over addition from the right.
theorem opposite_mul_add_distrib_right[R: Semiring](a: R, b: R, c: R) {
    opposite_mul(a + b, c) = opposite_mul(a, c) + opposite_mul(b, c)
} by {
    opposite_mul(a + b, c) = c * (a + b)
    c * (a + b) = c * a + c * b
    opposite_mul(a, c) = c * a
    opposite_mul(b, c) = c * b
}

/// A map preserving multiplication preserves opposite multiplication.
theorem preserves_opposite_mul[A: Mul, B: Mul](f: A -> B) {
    (forall(a: A, b: A) { f(a * b) = f(a) * f(b) }) implies
    forall(a: A, b: A) {
        f(opposite_mul(a, b)) = opposite_mul(f(a), f(b))
    }
} by {
    if forall(a: A, b: A) { f(a * b) = f(a) * f(b) } {
        forall(a: A, b: A) {
            opposite_mul(a, b) = b * a
            f(opposite_mul(a, b)) = f(b * a)
            f(b * a) = f(b) * f(a)
            opposite_mul(f(a), f(b)) = f(b) * f(a)
            f(opposite_mul(a, b)) = opposite_mul(f(a), f(b))
        }
    }
}

/// A map preserving addition and multiplication satisfies the left distributive law for opposite multiplication.
theorem opposite_mul_add_left_via_map[A: Semiring, B: Semiring](f: A -> B, a: A, b: A, c: A) {
    (forall(x: A, y: A) { f(x + y) = f(x) + f(y) }) and
    (forall(x: A, y: A) { f(x * y) = f(x) * f(y) }) implies
    f(opposite_mul(a, b + c)) = opposite_mul(f(a), f(b)) + opposite_mul(f(a), f(c))
} by {
    if (forall(x: A, y: A) { f(x + y) = f(x) + f(y) }) and
        (forall(x: A, y: A) { f(x * y) = f(x) * f(y) }) {
        opposite_mul(a, b + c) = (b + c) * a
        f(opposite_mul(a, b + c)) = f((b + c) * a)
        f((b + c) * a) = f(b + c) * f(a)
        f(b + c) = f(b) + f(c)
        f(opposite_mul(a, b + c)) = (f(b) + f(c)) * f(a)
        (f(b) + f(c)) * f(a) = f(b) * f(a) + f(c) * f(a)
        opposite_mul(f(a), f(b)) = f(b) * f(a)
        opposite_mul(f(a), f(c)) = f(c) * f(a)
        f(opposite_mul(a, b + c)) = opposite_mul(f(a), f(b)) + opposite_mul(f(a), f(c))
    }
}
