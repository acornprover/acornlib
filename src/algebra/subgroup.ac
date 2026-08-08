from nat import Nat, pow_zero
from algebra.group import Group, GroupHom, group_hom_one, group_hom_mul, group_hom_inv
from algebra.monoid.submonoid import Submonoid, submonoid_identity_constraint, submonoid_closure_constraint,
    submonoid_constraint
from data.basic.set import Set, empty_set_is_finite, singleton_contains_eq, singleton_set_is_finite,
    union_contains_eq, union_contains_left, union_contains_right,
    is_subset_monotone_map, is_set_extensive_map, is_set_idempotent_map,
    is_set_closure_operator
from data.basic.functions import predicate_extensionality

// We define subgroups with the "bundled subgroups" technique, so a subgroup "carries along"
// its group.

/// True if a subset contains the identity element.
define identity_constraint[G: Group](contains: G -> Bool) -> Bool {
    contains(G.1)
}

/// True if a subset is closed under the group operation.
define closure_constraint[G: Group](contains: G -> Bool) -> Bool {
    forall(a: G, b: G) {
        contains(a) and contains(b) implies contains(a * b)
    }
}

/// True if a subset is closed under taking inverses.
define inverse_constraint[G: Group](contains: G -> Bool) -> Bool {
    forall(a: G) {
        contains(a) implies contains(a.inverse)
    }
}

/// True if a subset satisfies all the requirements to be a subgroup.
define subgroup_constraint[G: Group](contains: G -> Bool) -> Bool {
    identity_constraint(contains) and closure_constraint(contains) and inverse_constraint(contains)
}

/// True if an element is the identity element of the group.
define is_identity[G: Group](g: G) -> Bool {
    g = G.1
}

theorem identity_subgroup_constraint[G: Group] {
    subgroup_constraint(is_identity[G])
} by {
    is_identity(G.1)
    identity_constraint(is_identity[G])
    forall(a: G, b: G) {
        if is_identity(a) and is_identity(b) {
            a = G.1
            b = G.1
            G.1 * G.1 = G.1
            is_identity(a * b)
        }
    }
    closure_constraint(is_identity[G])
    G.1.inverse = G.1
    forall(a: G) {
        is_identity(a) implies is_identity(a.inverse)
    }
    inverse_constraint(is_identity[G])
}

/// A subgroup of a group G, represented as a subset that is closed under the group operations.
structure Subgroup[G: Group] {
    /// True if the given element is a member of this subgroup.
    contains: G -> Bool
} constraint {
    subgroup_constraint(contains)
}

/// Subgroup extensionality: two subgroups are equal when they have the same elements.
theorem subgroup_ext[G: Group](a: Subgroup[G], b: Subgroup[G]) {
    (forall(x: G) { a.contains(x) = b.contains(x) }) implies a = b
} by {
    if forall(x: G) { a.contains(x) = b.contains(x) } {
        a.contains = b.contains
    }
}

/// Equal subgroups have equal membership predicates.
theorem subgroup_eq_contains[G: Group](a: Subgroup[G], b: Subgroup[G]) {
    a = b implies a.contains = b.contains
}

/// Equal subgroups have equal membership at every element.
theorem subgroup_eq_contains_at[G: Group](a: Subgroup[G], b: Subgroup[G], x: G) {
    a = b implies a.contains(x) = b.contains(x)
} by {
    if a = b {
        a.contains = b.contains
        a.contains(x) = b.contains(x)
    }
}

/// Equality of subgroups transports predicates on subgroups.
theorem subgroup_eq_transport_predicate[G: Group](
    p: Subgroup[G] -> Bool,
    a: Subgroup[G],
    b: Subgroup[G]
) {
    a = b and p(a) implies p(b)
}

/// Equality of subgroups transports predicates on subgroups in the reverse direction.
theorem subgroup_eq_transport_predicate_rev[G: Group](
    p: Subgroup[G] -> Bool,
    a: Subgroup[G],
    b: Subgroup[G]
) {
    a = b and p(b) implies p(a)
}

/// Equality of membership predicates determines equality of subgroups.
theorem subgroup_eq_of_contains_eq[G: Group](a: Subgroup[G], b: Subgroup[G]) {
    a.contains = b.contains implies a = b
} by {
    if a.contains = b.contains {
        forall(x: G) {
            a.contains(x) = b.contains(x)
        }
        subgroup_ext(a, b)
    }
}

/// Pointwise equality of membership determines equality of subgroups.
theorem subgroup_eq_of_contains_at_eq[G: Group](a: Subgroup[G], b: Subgroup[G]) {
    (forall(x: G) { a.contains(x) = b.contains(x) }) implies a = b
} by {
    if forall(x: G) { a.contains(x) = b.contains(x) } {
        subgroup_ext(a, b)
    }
}

/// A subgroup contains the identity element.
theorem subgroup_contains_identity[G: Group](s: Subgroup[G]) {
    s.contains(G.1)
} by {
    subgroup_constraint(s.contains)
    subgroup_constraint(s.contains) =
        identity_constraint(s.contains) and closure_constraint(s.contains) and inverse_constraint(s.contains)
    identity_constraint(s.contains)
}

/// A subgroup is closed under multiplication.
theorem subgroup_mul_mem[G: Group](s: Subgroup[G], a: G, b: G) {
    s.contains(a) and s.contains(b) implies s.contains(a * b)
} by {
    if s.contains(a) and s.contains(b) {
        subgroup_constraint(s.contains)
        subgroup_constraint(s.contains) =
            identity_constraint(s.contains) and closure_constraint(s.contains) and inverse_constraint(s.contains)
        closure_constraint(s.contains)
        closure_constraint(s.contains) = forall(x: G, y: G) {
            s.contains(x) and s.contains(y) implies s.contains(x * y)
        }
        s.contains(a * b)
    }
}

/// A subgroup is closed under inverse.
theorem subgroup_inv_mem[G: Group](s: Subgroup[G], a: G) {
    s.contains(a) implies s.contains(a.inverse)
} by {
    if s.contains(a) {
        subgroup_constraint(s.contains)
        subgroup_constraint(s.contains) =
            identity_constraint(s.contains) and closure_constraint(s.contains) and inverse_constraint(s.contains)
        inverse_constraint(s.contains)
        inverse_constraint(s.contains) = forall(x: G) {
            s.contains(x) implies s.contains(x.inverse)
        }
        s.contains(a.inverse)
    }
}

/// Membership is unchanged by taking inverses.
theorem subgroup_contains_inverse_iff[G: Group](s: Subgroup[G], a: G) {
    s.contains(a.inverse) = s.contains(a)
} by {
    if s.contains(a.inverse) {
        subgroup_inv_mem(s, a.inverse)
        s.contains(a.inverse.inverse)
        a.inverse.inverse = a
        s.contains(a)
    }
    if s.contains(a) {
        subgroup_inv_mem(s, a)
        s.contains(a.inverse)
    }
    s.contains(a.inverse) = s.contains(a)
}

/// A subgroup is closed under multiplying by an inverse on the right.
theorem subgroup_mul_inv_mem[G: Group](s: Subgroup[G], a: G, b: G) {
    s.contains(a) and s.contains(b) implies s.contains(a * b.inverse)
} by {
    if s.contains(a) and s.contains(b) {
        subgroup_inv_mem(s, b)
        s.contains(b.inverse)
        subgroup_mul_mem(s, a, b.inverse)
        s.contains(a * b.inverse)
    }
}

/// A subgroup is closed under multiplying by an inverse on the left.
theorem subgroup_inv_mul_mem[G: Group](s: Subgroup[G], a: G, b: G) {
    s.contains(a) and s.contains(b) implies s.contains(a.inverse * b)
} by {
    if s.contains(a) and s.contains(b) {
        subgroup_inv_mem(s, a)
        s.contains(a.inverse)
        subgroup_mul_mem(s, a.inverse, b)
        s.contains(a.inverse * b)
    }
}

/// Raw left-coset equivalence is reflexive.
theorem same_left_coset_raw_reflexive[G: Group](s: Subgroup[G], a: G) {
    s.contains(a.inverse * a)
} by {
    a.inverse * a = G.1
    subgroup_contains_identity(s)
}

/// Raw left-coset equivalence is symmetric.
theorem same_left_coset_raw_symmetric[G: Group](s: Subgroup[G], a: G, b: G) {
    s.contains(a.inverse * b) implies s.contains(b.inverse * a)
} by {
    s.contains((a.inverse * b).inverse)
    (a.inverse * b).inverse = b.inverse * a
    s.contains(b.inverse * a)
}

/// Raw left-coset equivalence is transitive.
theorem same_left_coset_raw_transitive[G: Group](s: Subgroup[G], a: G, b: G, c: G) {
    s.contains(a.inverse * b) and s.contains(b.inverse * c) implies s.contains(a.inverse * c)
} by {
    s.contains((a.inverse * b) * (b.inverse * c))
    (a.inverse * b) * (b.inverse * c) = a.inverse * c
    s.contains(a.inverse * c)
}

/// An intersection witness shows that two raw left-coset representatives are equivalent.
theorem left_coset_intersection_witness_implies_same_representative_raw[G: Group](
    s: Subgroup[G],
    a: G,
    b: G,
    x: G
) {
    s.contains(a.inverse * x) and s.contains(b.inverse * x) implies s.contains(a.inverse * b)
} by {
    same_left_coset_raw_symmetric(s, b, x)
    s.contains(x.inverse * b)
    same_left_coset_raw_transitive(s, a, x, b)
}

/// Equivalent raw left-coset representatives have the same membership predicate.
theorem same_representative_implies_left_coset_membership_eq_raw[G: Group](
    s: Subgroup[G],
    a: G,
    b: G,
    y: G
) {
    s.contains(a.inverse * b) implies s.contains(a.inverse * y) = s.contains(b.inverse * y)
} by {
    if s.contains(a.inverse * y) {
        same_left_coset_raw_symmetric(s, a, b)
        s.contains(b.inverse * a)
        same_left_coset_raw_transitive(s, b, a, y)
        s.contains(b.inverse * y)
    }
    if s.contains(b.inverse * y) {
        same_left_coset_raw_transitive(s, a, b, y)
        s.contains(a.inverse * y)
    }
}

/// Intersecting raw left cosets are equal as membership predicates.
theorem intersecting_left_cosets_equal_raw[G: Group](s: Subgroup[G], a: G, b: G) {
    exists(x: G) {
        s.contains(a.inverse * x) and s.contains(b.inverse * x)
    } implies forall(y: G) {
        s.contains(a.inverse * y) = s.contains(b.inverse * y)
    }
} by {
    let x: G satisfy { s.contains(a.inverse * x) and s.contains(b.inverse * x) }
    left_coset_intersection_witness_implies_same_representative_raw(s, a, b, x)
    s.contains(a.inverse * b)
    forall(y: G) {
        same_representative_implies_left_coset_membership_eq_raw(s, a, b, y)
        s.contains(a.inverse * y) = s.contains(b.inverse * y)
    }
}

/// A subgroup is closed under products of three of its elements.
theorem subgroup_mul_mul_mem[G: Group](s: Subgroup[G], a: G, b: G, c: G) {
    s.contains(a) and s.contains(b) and s.contains(c) implies s.contains(a * b * c)
} by {
    if s.contains(a) and s.contains(b) and s.contains(c) {
        subgroup_mul_mem(s, a, b)
        s.contains(a * b)
        subgroup_mul_mem(s, a * b, c)
        s.contains(a * b * c)
    }
}

/// A subgroup is closed under conjugation by one of its elements.
theorem subgroup_conj_mem[G: Group](s: Subgroup[G], a: G, b: G) {
    s.contains(a) and s.contains(b) implies s.contains(a * b * a.inverse)
} by {
    if s.contains(a) and s.contains(b) {
        subgroup_inv_mem(s, a)
        s.contains(a.inverse)
        subgroup_mul_mul_mem(s, a, b, a.inverse)
        s.contains(a * b * a.inverse)
    }
}

/// A subgroup contains every natural power of each of its elements.
theorem subgroup_pow_mem[G: Group](s: Subgroup[G], a: G, n: Nat) {
    s.contains(a) implies s.contains(a.pow(n))
} by {
    define p(k: Nat) -> Bool {
        s.contains(a) implies s.contains(a.pow(k))
    }

    if s.contains(a) {
        a.pow(Nat.0) = G.1
        subgroup_contains_identity(s)
        s.contains(G.1)
        s.contains(a.pow(Nat.0))
    }
    p(Nat.0)

    forall(k: Nat) {
        if p(k) {
            if s.contains(a) {
                s.contains(a.pow(k))
                subgroup_mul_mem(s, a, a.pow(k))
                s.contains(a * a.pow(k))
                a.pow(k.suc) = a * a.pow(k)
                s.contains(a.pow(k.suc))
            }
            p(k.suc)
        }
    }
    p(n)
    if s.contains(a) {
        s.contains(a.pow(n))
    }
}

/// The common membership predicate of two subgroups.
define subgroup_intersection_contains[G: Group](a: Subgroup[G], b: Subgroup[G], x: G) -> Bool {
    a.contains(x) and b.contains(x)
}

/// The common part of two subgroups contains the identity element.
theorem subgroup_intersection_identity_constraint[G: Group](a: Subgroup[G], b: Subgroup[G]) {
    identity_constraint(subgroup_intersection_contains(a, b))
} by {
    subgroup_contains_identity(a)
    subgroup_contains_identity(b)
    subgroup_intersection_contains(a, b, G.1)
}

/// The common part of two subgroups is closed under multiplication.
theorem subgroup_intersection_closure_constraint[G: Group](a: Subgroup[G], b: Subgroup[G]) {
    closure_constraint(subgroup_intersection_contains(a, b))
} by {
    forall(x: G, y: G) {
        if subgroup_intersection_contains(a, b, x) and subgroup_intersection_contains(a, b, y) {
            a.contains(x)
            b.contains(x)
            a.contains(y)
            b.contains(y)
            subgroup_mul_mem(a, x, y)
            subgroup_mul_mem(b, x, y)
            a.contains(x * y)
            b.contains(x * y)
            subgroup_intersection_contains(a, b, x * y)
        }
    }
}

/// The common part of two subgroups is closed under inverse.
theorem subgroup_intersection_inverse_constraint[G: Group](a: Subgroup[G], b: Subgroup[G]) {
    inverse_constraint(subgroup_intersection_contains(a, b))
} by {
    forall(x: G) {
        if subgroup_intersection_contains(a, b, x) {
            a.contains(x)
            b.contains(x)
            subgroup_inv_mem(a, x)
            subgroup_inv_mem(b, x)
            a.contains(x.inverse)
            b.contains(x.inverse)
            subgroup_intersection_contains(a, b, x.inverse)
        }
    }
}

/// The common part of two subgroups is a subgroup.
theorem subgroup_intersection_constraint[G: Group](a: Subgroup[G], b: Subgroup[G]) {
    subgroup_constraint(subgroup_intersection_contains(a, b))
} by {
    subgroup_intersection_identity_constraint(a, b)
    subgroup_intersection_closure_constraint(a, b)
    subgroup_intersection_inverse_constraint(a, b)
}

let subgroup_intersection[G: Group](a: Subgroup[G], b: Subgroup[G]) -> result: Subgroup[G] satisfy {
    Subgroup.new(subgroup_intersection_contains(a, b)) = Option.some(result)
} by {
    subgroup_intersection_constraint(a, b)
}

attributes Subgroup[G: Group] {
    /// Subgroup extensionality from pointwise equality of membership.
    let ext = subgroup_ext[G]

    /// The subset of group elements belonging to this subgroup.
    define as_set(self) -> Set[G] {
        Set[G].new(self.contains)
    }

    /// The common part of two subgroups.
    let intersection: (Subgroup[G], Subgroup[G]) -> Subgroup[G] = subgroup_intersection
}

/// Membership in the underlying set is membership in the subgroup.
theorem subgroup_as_set_contains_eq[G: Group](s: Subgroup[G], x: G) {
    s.as_set.contains(x) = s.contains(x)
} by {
    s.as_set.contains(x) = s.contains(x)
}

/// Membership in a subgroup is membership in its underlying set.
theorem subgroup_contains_as_set_eq[G: Group](s: Subgroup[G], x: G) {
    s.contains(x) = s.as_set.contains(x)
} by {
    subgroup_as_set_contains_eq(s, x)
    s.contains(x) = s.as_set.contains(x)
}

/// Equal subgroups have equal underlying sets.
theorem subgroup_eq_as_set[G: Group](a: Subgroup[G], b: Subgroup[G]) {
    a = b implies a.as_set = b.as_set
}

/// Membership in the intersection means membership in both subgroups.
theorem subgroup_intersection_contains_eq[G: Group](a: Subgroup[G], b: Subgroup[G], x: G) {
    a.intersection(b).contains(x) = (a.contains(x) and b.contains(x))
} by {
    a.intersection(b).contains(x) = subgroup_intersection_contains(a, b, x)
    subgroup_intersection_contains(a, b, x) = (a.contains(x) and b.contains(x))
}

/// The intersection is contained in the left subgroup.
theorem subgroup_intersection_subset_left[G: Group](a: Subgroup[G], b: Subgroup[G], x: G) {
    a.intersection(b).contains(x) implies a.contains(x)
} by {
    if a.intersection(b).contains(x) {
        subgroup_intersection_contains_eq(a, b, x)
        a.contains(x)
    }
}

/// The intersection is contained in the right subgroup.
theorem subgroup_intersection_subset_right[G: Group](a: Subgroup[G], b: Subgroup[G], x: G) {
    a.intersection(b).contains(x) implies b.contains(x)
} by {
    if a.intersection(b).contains(x) {
        subgroup_intersection_contains_eq(a, b, x)
        b.contains(x)
    }
}

/// Subgroup intersection is commutative.
theorem subgroup_intersection_comm[G: Group](a: Subgroup[G], b: Subgroup[G]) {
    a.intersection(b) = b.intersection(a)
} by {
    forall(x: G) {
        if a.intersection(b).contains(x) {
            subgroup_intersection_contains_eq(a, b, x)
            a.contains(x)
            b.contains(x)
            subgroup_intersection_contains_eq(b, a, x)
            b.intersection(a).contains(x)
        }
        if b.intersection(a).contains(x) {
            subgroup_intersection_contains_eq(b, a, x)
            b.contains(x)
            a.contains(x)
            subgroup_intersection_contains_eq(a, b, x)
            a.intersection(b).contains(x)
        }
        a.intersection(b).contains(x) = b.intersection(a).contains(x)
    }
    subgroup_ext(a.intersection(b), b.intersection(a))
}

/// Subgroup intersection is associative.
theorem subgroup_intersection_assoc[G: Group](
    a: Subgroup[G],
    b: Subgroup[G],
    c: Subgroup[G]
) {
    a.intersection(b).intersection(c) = a.intersection(b.intersection(c))
} by {
    let lhs = a.intersection(b).intersection(c)
    let rhs = a.intersection(b.intersection(c))
    forall(x: G) {
        if lhs.contains(x) {
            subgroup_intersection_contains_eq(a.intersection(b), c, x)
            a.intersection(b).contains(x)
            c.contains(x)
            subgroup_intersection_contains_eq(a, b, x)
            a.contains(x)
            b.contains(x)
            subgroup_intersection_contains_eq(b, c, x)
            b.intersection(c).contains(x)
            subgroup_intersection_contains_eq(a, b.intersection(c), x)
            rhs.contains(x)
        }
        if rhs.contains(x) {
            subgroup_intersection_contains_eq(a, b.intersection(c), x)
            a.contains(x)
            b.intersection(c).contains(x)
            subgroup_intersection_contains_eq(b, c, x)
            b.contains(x)
            c.contains(x)
            subgroup_intersection_contains_eq(a, b, x)
            a.intersection(b).contains(x)
            subgroup_intersection_contains_eq(a.intersection(b), c, x)
            lhs.contains(x)
        }
        lhs.contains(x) = rhs.contains(x)
    }
    subgroup_ext(lhs, rhs)
}

/// Subgroup intersection is idempotent.
theorem subgroup_intersection_idempotent[G: Group](a: Subgroup[G]) {
    a.intersection(a) = a
} by {
    forall(x: G) {
        if a.intersection(a).contains(x) {
            subgroup_intersection_contains_eq(a, a, x)
            a.contains(x)
        }
        if a.contains(x) {
            subgroup_intersection_contains_eq(a, a, x)
            a.intersection(a).contains(x)
        }
        a.intersection(a).contains(x) = a.contains(x)
    }
    subgroup_ext(a.intersection(a), a)
}

/// True if every element of one subgroup belongs to another.
define subgroup_subset[G: Group](a: Subgroup[G], b: Subgroup[G]) -> Bool {
    forall(x: G) {
        a.contains(x) implies b.contains(x)
    }
}

/// Subgroup containment is containment of the underlying sets.
theorem subgroup_subset_as_set_eq[G: Group](a: Subgroup[G], b: Subgroup[G]) {
    subgroup_subset(a, b) = a.as_set.subset(b.as_set)
} by {
    if subgroup_subset(a, b) {
        forall(x: G) {
            if a.as_set.contains(x) {
                subgroup_as_set_contains_eq(a, x)
                a.contains(x)
                b.contains(x)
                subgroup_as_set_contains_eq(b, x)
                b.as_set.contains(x)
            }
        }
        a.as_set.subset(b.as_set)
    }
    if a.as_set.subset(b.as_set) {
        forall(x: G) {
            if a.contains(x) {
                subgroup_as_set_contains_eq(a, x)
                a.as_set.contains(x)
                a.as_set.subset(b.as_set) = forall(y: G) {
                    a.as_set.contains(y) implies b.as_set.contains(y)
                }
                b.as_set.contains(x)
                subgroup_as_set_contains_eq(b, x)
                b.contains(x)
            }
        }
        subgroup_subset(a, b)
    }
    subgroup_subset(a, b) = a.as_set.subset(b.as_set)
}

/// Subgroup containment gives containment of underlying sets.
theorem subgroup_as_set_subset_of_subset[G: Group](a: Subgroup[G], b: Subgroup[G]) {
    subgroup_subset(a, b) implies a.as_set.subset(b.as_set)
} by {
    if subgroup_subset(a, b) {
        subgroup_subset_as_set_eq(a, b)
        a.as_set.subset(b.as_set)
    }
}

/// Containment of underlying sets gives subgroup containment.
theorem subgroup_subset_of_as_set_subset[G: Group](a: Subgroup[G], b: Subgroup[G]) {
    a.as_set.subset(b.as_set) implies subgroup_subset(a, b)
} by {
    if a.as_set.subset(b.as_set) {
        subgroup_subset_as_set_eq(a, b)
        subgroup_subset(a, b)
    }
}

/// Subgroup inclusion is reflexive.
theorem subgroup_subset_refl[G: Group](a: Subgroup[G]) {
    subgroup_subset(a, a)
} by {
    forall(x: G) {
        a.contains(x) implies a.contains(x)
    }
}

/// Subgroup inclusion is transitive.
theorem subgroup_subset_trans[G: Group](a: Subgroup[G], b: Subgroup[G], c: Subgroup[G]) {
    subgroup_subset(a, b) and subgroup_subset(b, c) implies subgroup_subset(a, c)
} by {
    if subgroup_subset(a, b) and subgroup_subset(b, c) {
        forall(x: G) {
            if a.contains(x) {
                b.contains(x)
                c.contains(x)
            }
        }
    }
}

/// The intersection of two subgroups is contained in the left subgroup.
theorem subgroup_intersection_subset_left_relation[G: Group](a: Subgroup[G], b: Subgroup[G]) {
    subgroup_subset(a.intersection(b), a)
} by {
    forall(x: G) {
        if a.intersection(b).contains(x) {
            subgroup_intersection_contains_eq(a, b, x)
            a.contains(x)
        }
    }
}

/// The intersection of two subgroups is contained in the right subgroup.
theorem subgroup_intersection_subset_right_relation[G: Group](a: Subgroup[G], b: Subgroup[G]) {
    subgroup_subset(a.intersection(b), b)
} by {
    forall(x: G) {
        if a.intersection(b).contains(x) {
            subgroup_intersection_contains_eq(a, b, x)
            b.contains(x)
        }
    }
}

/// Every common subgroup of two subgroups is contained in their intersection.
theorem subgroup_subset_intersection_of_subset_left_right[G: Group](
    c: Subgroup[G], a: Subgroup[G], b: Subgroup[G]
) {
    subgroup_subset(c, a) and subgroup_subset(c, b) implies subgroup_subset(c, a.intersection(b))
} by {
    if subgroup_subset(c, a) and subgroup_subset(c, b) {
        forall(x: G) {
            if c.contains(x) {
                a.contains(x)
                b.contains(x)
                subgroup_intersection_contains_eq(a, b, x)
                a.intersection(b).contains(x)
            }
        }
    }
}

/// Containment in an intersection is equivalent to containment in both subgroups.
theorem subgroup_subset_intersection_iff[G: Group](c: Subgroup[G], a: Subgroup[G], b: Subgroup[G]) {
    subgroup_subset(c, a.intersection(b)) = (subgroup_subset(c, a) and subgroup_subset(c, b))
} by {
    if subgroup_subset(c, a.intersection(b)) {
        subgroup_intersection_subset_left_relation(a, b)
        subgroup_subset_trans(c, a.intersection(b), a)
        subgroup_subset(c, a)
        subgroup_intersection_subset_right_relation(a, b)
        subgroup_subset_trans(c, a.intersection(b), b)
        subgroup_subset(c, b)
        subgroup_subset(c, a) and subgroup_subset(c, b)
    }
    if subgroup_subset(c, a) and subgroup_subset(c, b) {
        subgroup_subset_intersection_of_subset_left_right(c, a, b)
        subgroup_subset(c, a.intersection(b))
    }
    subgroup_subset(c, a.intersection(b)) = (subgroup_subset(c, a) and subgroup_subset(c, b))
}

/// The trivial subgroup containing only the identity element.
let identity_subgroup[G: Group]: Subgroup[G] satisfy {
    Subgroup.new(is_identity[G]) = Option.some(identity_subgroup)
}

theorem identity_subgroup_only_has_identity[G: Group](g: G) {
    identity_subgroup[G].contains(g) implies g = G.1
}

/// Membership in the identity subgroup is the same as equality to the identity element.
theorem identity_subgroup_contains_eq[G: Group](g: G) {
    identity_subgroup[G].contains(g) = (g = G.1)
} by {
    if identity_subgroup[G].contains(g) {
        identity_subgroup_only_has_identity(g)
        g = G.1
    }
    if g = G.1 {
        subgroup_contains_identity(identity_subgroup[G])
        identity_subgroup[G].contains(G.1)
        identity_subgroup[G].contains(g)
    }
    identity_subgroup[G].contains(g) = (g = G.1)
}

/// The identity subgroup is contained in every subgroup.
theorem identity_subgroup_subset[G: Group](s: Subgroup[G]) {
    subgroup_subset(identity_subgroup[G], s)
} by {
    forall(x: G) {
        if identity_subgroup[G].contains(x) {
            identity_subgroup_only_has_identity(x)
            x = G.1
            subgroup_contains_identity(s)
            s.contains(G.1)
            s.contains(x)
        }
    }
}

/// The full subset of any group is a subgroup.
define full_group_contains[G: Group](a: G) -> Bool {
    true
}

/// The full membership predicate is a subgroup predicate.
theorem full_subgroup_constraint[G: Group] {
    subgroup_constraint(full_group_contains[G])
} by {
    full_group_contains[G](G.1)
    identity_constraint(full_group_contains[G])
    forall(a: G, b: G) {
        if full_group_contains[G](a) and full_group_contains[G](b) {
            full_group_contains[G](a * b)
        }
    }
    closure_constraint(full_group_contains[G])
    forall(a: G) {
        if full_group_contains[G](a) {
            full_group_contains[G](a.inverse)
        }
    }
    inverse_constraint(full_group_contains[G])
}

/// The full subgroup, containing every element.
let full_subgroup[G: Group]: Subgroup[G] satisfy {
    Subgroup.new(full_group_contains[G]) = Option.some(full_subgroup)
}

/// Every group element belongs to the full subgroup.
theorem full_subgroup_contains_everything[G: Group](g: G) {
    full_subgroup[G].contains(g)
}

/// Membership in the full subgroup is always true.
theorem full_subgroup_contains_eq[G: Group](g: G) {
    full_subgroup[G].contains(g) = true
} by {
    full_subgroup_contains_everything(g)
}

/// Every subgroup is contained in the full subgroup.
theorem subgroup_subset_full[G: Group](s: Subgroup[G]) {
    subgroup_subset(s, full_subgroup[G])
} by {
    forall(x: G) {
        if s.contains(x) {
            full_subgroup_contains_everything(x)
        }
    }
}

/// Mutual inclusion of subgroups forces equality.
theorem subgroup_subset_antisymm[G: Group](a: Subgroup[G], b: Subgroup[G]) {
    subgroup_subset(a, b) and subgroup_subset(b, a) implies a = b
} by {
    if subgroup_subset(a, b) and subgroup_subset(b, a) {
        subgroup_subset(a, b) = forall(y: G) {
            a.contains(y) implies b.contains(y)
        }
        subgroup_subset(b, a) = forall(y: G) {
            b.contains(y) implies a.contains(y)
        }
        forall(x: G) {
            if a.contains(x) {
                b.contains(x)
            }
            if b.contains(x) {
                a.contains(x)
            }
            a.contains(x) = b.contains(x)
        }
        predicate_extensionality(a.contains, b.contains)
        a.contains = b.contains
        a = b
    }
}

/// Equality of subgroups is equivalent to mutual inclusion.
theorem subgroup_eq_iff_subset_both[G: Group](a: Subgroup[G], b: Subgroup[G]) {
    a = b = (subgroup_subset(a, b) and subgroup_subset(b, a))
} by {
    if a = b {
        subgroup_subset_refl(a)
        subgroup_subset(a, b)
        subgroup_subset(b, a)
        subgroup_subset(a, b) and subgroup_subset(b, a)
    }
    if subgroup_subset(a, b) and subgroup_subset(b, a) {
        subgroup_subset_antisymm(a, b)
        a = b
    }
    a = b = (subgroup_subset(a, b) and subgroup_subset(b, a))
}

/// Intersecting with a larger subgroup gives the smaller subgroup.
theorem subgroup_intersection_eq_left_of_subset[G: Group](a: Subgroup[G], b: Subgroup[G]) {
    subgroup_subset(a, b) implies a.intersection(b) = a
} by {
    if subgroup_subset(a, b) {
        subgroup_intersection_subset_left_relation(a, b)
        subgroup_subset_intersection_of_subset_left_right(a, a, b)
        subgroup_subset(a, a.intersection(b))
        subgroup_subset_antisymm(a.intersection(b), a)
        a.intersection(b) = a
    }
}

/// Intersecting with a larger subgroup on the left gives the smaller subgroup.
theorem subgroup_intersection_eq_right_of_subset[G: Group](a: Subgroup[G], b: Subgroup[G]) {
    subgroup_subset(b, a) implies a.intersection(b) = b
} by {
    if subgroup_subset(b, a) {
        subgroup_intersection_subset_right_relation(a, b)
        subgroup_subset_intersection_of_subset_left_right(b, a, b)
        subgroup_subset(b, a.intersection(b))
        subgroup_subset_antisymm(a.intersection(b), b)
        a.intersection(b) = b
    }
}

/// Intersecting the identity subgroup on the left gives the identity subgroup.
theorem identity_subgroup_intersection_left[G: Group](s: Subgroup[G]) {
    identity_subgroup[G].intersection(s) = identity_subgroup[G]
} by {
    identity_subgroup_subset(s)
    subgroup_intersection_eq_left_of_subset(identity_subgroup[G], s)
}

/// Intersecting the identity subgroup on the right gives the identity subgroup.
theorem identity_subgroup_intersection_right[G: Group](s: Subgroup[G]) {
    s.intersection(identity_subgroup[G]) = identity_subgroup[G]
} by {
    identity_subgroup_subset(s)
    subgroup_intersection_eq_right_of_subset(s, identity_subgroup[G])
}

/// Intersecting the full subgroup on the left gives the other subgroup.
theorem full_subgroup_intersection_left[G: Group](s: Subgroup[G]) {
    full_subgroup[G].intersection(s) = s
} by {
    subgroup_subset_full(s)
    subgroup_intersection_eq_right_of_subset(full_subgroup[G], s)
}

/// Intersecting the full subgroup on the right gives the other subgroup.
theorem full_subgroup_intersection_right[G: Group](s: Subgroup[G]) {
    s.intersection(full_subgroup[G]) = s
} by {
    subgroup_subset_full(s)
    subgroup_intersection_eq_left_of_subset(s, full_subgroup[G])
}

/// True if every element of a set belongs to a subgroup.
define set_subset_subgroup[G: Group](a: Set[G], s: Subgroup[G]) -> Bool {
    forall(x: G) {
        a.contains(x) implies s.contains(x)
    }
}

/// Set containment in a subgroup is containment in its underlying set.
theorem set_subset_subgroup_as_set_eq[G: Group](a: Set[G], s: Subgroup[G]) {
    set_subset_subgroup(a, s) = a.subset(s.as_set)
} by {
    if set_subset_subgroup(a, s) {
        forall(x: G) {
            if a.contains(x) {
                s.contains(x)
                subgroup_as_set_contains_eq(s, x)
                s.as_set.contains(x)
            }
        }
        a.subset(s.as_set)
    }
    if a.subset(s.as_set) {
        forall(x: G) {
            if a.contains(x) {
                a.subset(s.as_set) = forall(y: G) {
                    a.contains(y) implies s.as_set.contains(y)
                }
                s.as_set.contains(x)
                subgroup_as_set_contains_eq(s, x)
                s.contains(x)
            }
        }
        set_subset_subgroup(a, s)
    }
    set_subset_subgroup(a, s) = a.subset(s.as_set)
}

/// Set containment in a subgroup gives containment in its underlying set.
theorem set_as_set_subset_of_subset_subgroup[G: Group](a: Set[G], s: Subgroup[G]) {
    set_subset_subgroup(a, s) implies a.subset(s.as_set)
} by {
    if set_subset_subgroup(a, s) {
        set_subset_subgroup_as_set_eq(a, s)
        a.subset(s.as_set)
    }
}

/// Containment in the underlying set gives set containment in the subgroup.
theorem set_subset_subgroup_of_as_set_subset[G: Group](a: Set[G], s: Subgroup[G]) {
    a.subset(s.as_set) implies set_subset_subgroup(a, s)
} by {
    if a.subset(s.as_set) {
        set_subset_subgroup_as_set_eq(a, s)
        set_subset_subgroup(a, s)
    }
}

/// The underlying set of a subgroup is contained in the subgroup.
theorem subgroup_as_set_subset_self[G: Group](s: Subgroup[G]) {
    set_subset_subgroup(s.as_set, s)
} by {
    forall(x: G) {
        if s.as_set.contains(x) {
            s.contains(x)
        }
    }
}

/// Set containment is preserved by enlarging the target subgroup.
theorem set_subset_subgroup_trans[G: Group](
    a: Set[G],
    s: Subgroup[G],
    t: Subgroup[G]
) {
    set_subset_subgroup(a, s) and subgroup_subset(s, t) implies set_subset_subgroup(a, t)
} by {
    if set_subset_subgroup(a, s) and subgroup_subset(s, t) {
        forall(x: G) {
            if a.contains(x) {
                s.contains(x)
                t.contains(x)
            }
        }
    }
}

/// Set containment in a subgroup is preserved by enlarging the set.
theorem set_subset_subgroup_of_set_subset[G: Group](
    a: Set[G],
    b: Set[G],
    s: Subgroup[G]
) {
    a.subset(b) and set_subset_subgroup(b, s) implies set_subset_subgroup(a, s)
} by {
    if a.subset(b) and set_subset_subgroup(b, s) {
        forall(x: G) {
            if a.contains(x) {
                a.subset(b) = forall(y: G) {
                    a.contains(y) implies b.contains(y)
                }
                b.contains(x)
                s.contains(x)
            }
        }
    }
}

/// True if an element belongs to every subgroup that contains a given set.
define subgroup_closure_contains[G: Group](a: Set[G], x: G) -> Bool {
    forall(s: Subgroup[G]) {
        set_subset_subgroup(a, s) implies s.contains(x)
    }
}

/// Membership in the closure gives membership in each subgroup that contains the set.
theorem subgroup_closure_contains_of_set_subset_raw[G: Group](
    a: Set[G],
    s: Subgroup[G],
    x: G
) {
    subgroup_closure_contains(a, x) and set_subset_subgroup(a, s) implies s.contains(x)
} by {
    if subgroup_closure_contains(a, x) and set_subset_subgroup(a, s) {
        subgroup_closure_contains(a, x) = forall(t: Subgroup[G]) {
            set_subset_subgroup(a, t) implies t.contains(x)
        }
        s.contains(x)
    }
}

/// The subgroup closure of a set contains the identity.
theorem subgroup_closure_identity_constraint[G: Group](a: Set[G]) {
    identity_constraint(subgroup_closure_contains(a))
} by {
    forall(s: Subgroup[G]) {
        if set_subset_subgroup(a, s) {
            subgroup_contains_identity(s)
            s.contains(G.1)
        }
    }
    subgroup_closure_contains(a, G.1)
    identity_constraint(subgroup_closure_contains(a))
}

/// The subgroup closure of a set is closed under multiplication.
theorem subgroup_closure_closure_constraint[G: Group](a: Set[G]) {
    closure_constraint(subgroup_closure_contains(a))
} by {
    forall(x: G, y: G) {
        if subgroup_closure_contains(a, x) and subgroup_closure_contains(a, y) {
            forall(s: Subgroup[G]) {
                if set_subset_subgroup(a, s) {
                    subgroup_closure_contains_of_set_subset_raw(a, s, x)
                    s.contains(x)
                    subgroup_closure_contains_of_set_subset_raw(a, s, y)
                    s.contains(y)
                    subgroup_mul_mem(s, x, y)
                    s.contains(x * y)
                }
            }
            subgroup_closure_contains(a, x * y)
        }
    }
}

/// The subgroup closure of a set is closed under inverses.
theorem subgroup_closure_inverse_constraint[G: Group](a: Set[G]) {
    inverse_constraint(subgroup_closure_contains(a))
} by {
    forall(x: G) {
        if subgroup_closure_contains(a, x) {
            forall(s: Subgroup[G]) {
                if set_subset_subgroup(a, s) {
                    subgroup_closure_contains_of_set_subset_raw(a, s, x)
                    s.contains(x)
                    subgroup_inv_mem(s, x)
                    s.contains(x.inverse)
                }
            }
            subgroup_closure_contains(a, x.inverse)
        }
    }
}

/// The subgroup generated by a set satisfies the subgroup laws.
theorem subgroup_generated_constraint[G: Group](a: Set[G]) {
    subgroup_constraint(subgroup_closure_contains(a))
} by {
    subgroup_closure_identity_constraint(a)
    subgroup_closure_closure_constraint(a)
    subgroup_closure_inverse_constraint(a)
}

/// The smallest subgroup containing a set.
let subgroup_closure[G: Group](a: Set[G]) -> result: Subgroup[G] satisfy {
    Subgroup.new(subgroup_closure_contains(a)) = Option.some(result)
} by {
    subgroup_generated_constraint(a)
}

/// Membership in the closure means membership in every subgroup containing the set.
theorem subgroup_closure_contains_eq[G: Group](a: Set[G], x: G) {
    subgroup_closure(a).contains(x) = subgroup_closure_contains(a, x)
} by {
    subgroup_closure(a).contains(x) = subgroup_closure_contains(a, x)
}

/// Every generator belongs to the subgroup closure.
theorem subgroup_subset_closure[G: Group](a: Set[G]) {
    set_subset_subgroup(a, subgroup_closure(a))
} by {
    forall(x: G) {
        if a.contains(x) {
            forall(s: Subgroup[G]) {
                if set_subset_subgroup(a, s) {
                    s.contains(x)
                }
            }
            subgroup_closure_contains(a, x)
            subgroup_closure_contains_eq(a, x)
            subgroup_closure(a).contains(x)
        }
    }
}

/// A generator is a member of the subgroup closure.
theorem subgroup_closure_contains_of_set_contains[G: Group](a: Set[G], x: G) {
    a.contains(x) implies subgroup_closure(a).contains(x)
} by {
    if a.contains(x) {
        subgroup_subset_closure(a)
        subgroup_closure(a).contains(x)
    }
}

/// The subgroup closure is contained in any subgroup containing the set.
theorem subgroup_closure_subset_of_set_subset[G: Group](a: Set[G], s: Subgroup[G]) {
    set_subset_subgroup(a, s) implies subgroup_subset(subgroup_closure(a), s)
} by {
    if set_subset_subgroup(a, s) {
        forall(x: G) {
            if subgroup_closure(a).contains(x) {
                subgroup_closure_contains_eq(a, x)
                subgroup_closure_contains(a, x)
                subgroup_closure_contains_of_set_subset_raw(a, s, x)
            }
        }
    }
}

/// The subgroup closure is the least subgroup containing the set.
theorem subgroup_closure_le_iff_set_subset[G: Group](a: Set[G], s: Subgroup[G]) {
    subgroup_subset(subgroup_closure(a), s) = set_subset_subgroup(a, s)
} by {
    if subgroup_subset(subgroup_closure(a), s) {
        subgroup_subset_closure(a)
        set_subset_subgroup_trans(a, subgroup_closure(a), s)
        set_subset_subgroup(a, s)
    }
    if set_subset_subgroup(a, s) {
        subgroup_closure_subset_of_set_subset(a, s)
        subgroup_subset(subgroup_closure(a), s)
    }
    subgroup_subset(subgroup_closure(a), s) = set_subset_subgroup(a, s)
}

/// The subgroup closure and underlying set maps form a set-containment adjunction.
theorem subgroup_closure_as_set_galois_connection[G: Group](a: Set[G], s: Subgroup[G]) {
    subgroup_closure(a).as_set.subset(s.as_set) = a.subset(s.as_set)
} by {
    subgroup_subset_as_set_eq(subgroup_closure(a), s)
    subgroup_closure_le_iff_set_subset(a, s)
    set_subset_subgroup_as_set_eq(a, s)
    subgroup_closure(a).as_set.subset(s.as_set) = a.subset(s.as_set)
}

/// The closure of the underlying set of a subgroup is the subgroup itself.
theorem subgroup_closure_as_set[G: Group](s: Subgroup[G]) {
    subgroup_closure(s.as_set) = s
} by {
    subgroup_as_set_subset_self(s)
    subgroup_closure_subset_of_set_subset(s.as_set, s)
    subgroup_subset(subgroup_closure(s.as_set), s)
    subgroup_subset_closure(s.as_set)
    forall(x: G) {
        if s.contains(x) {
            s.as_set.contains(x)
            subgroup_closure(s.as_set).contains(x)
        }
    }
    subgroup_subset(s, subgroup_closure(s.as_set))
    subgroup_subset_antisymm(subgroup_closure(s.as_set), s)
}

/// Subgroup closure is monotone with respect to set inclusion.
theorem subgroup_closure_mono[G: Group](a: Set[G], b: Set[G]) {
    a.subset(b) implies subgroup_subset(subgroup_closure(a), subgroup_closure(b))
} by {
    if a.subset(b) {
        subgroup_subset_closure(b)
        set_subset_subgroup_of_set_subset(a, b, subgroup_closure(b))
        set_subset_subgroup(a, subgroup_closure(b))
        subgroup_closure_subset_of_set_subset(a, subgroup_closure(b))
        subgroup_subset(subgroup_closure(a), subgroup_closure(b))
    }
}

/// Equal sets have equal subgroup closures.
theorem subgroup_closure_eq_of_set_eq[G: Group](a: Set[G], b: Set[G]) {
    a = b implies subgroup_closure(a) = subgroup_closure(b)
} by {
    if a = b {
        subgroup_closure(a) = subgroup_closure(b)
    }
}

/// Applying subgroup closure twice gives the same subgroup.
theorem subgroup_closure_idempotent[G: Group](a: Set[G]) {
    subgroup_closure(subgroup_closure(a).as_set) = subgroup_closure(a)
} by {
    subgroup_closure_as_set(subgroup_closure(a))
}

/// The set closure induced by subgroup generation.
define subgroup_set_closure[G: Group](a: Set[G]) -> Set[G] {
    subgroup_closure(a).as_set
}

/// The subgroup set closure is the underlying set of the generated subgroup.
theorem subgroup_set_closure_at[G: Group](a: Set[G]) {
    subgroup_set_closure(a) = subgroup_closure(a).as_set
}

/// The subgroup set closure contains the original set.
theorem subgroup_set_closure_extensive[G: Group](a: Set[G]) {
    a.subset(subgroup_set_closure(a))
} by {
    subgroup_set_closure_at(a)
    subgroup_subset_closure(a)
    set_subset_subgroup_as_set_eq(a, subgroup_closure(a))
    a.subset(subgroup_closure(a).as_set)
    a.subset(subgroup_set_closure(a))
}

/// The subgroup set closure preserves inclusion.
theorem subgroup_set_closure_mono[G: Group](a: Set[G], b: Set[G]) {
    a.subset(b) implies subgroup_set_closure(a).subset(subgroup_set_closure(b))
} by {
    if a.subset(b) {
        subgroup_closure_mono(a, b)
        subgroup_subset(subgroup_closure(a), subgroup_closure(b))
        subgroup_subset_as_set_eq(subgroup_closure(a), subgroup_closure(b))
        subgroup_closure(a).as_set.subset(subgroup_closure(b).as_set)
        subgroup_set_closure_at(a)
        subgroup_set_closure_at(b)
        subgroup_set_closure(a).subset(subgroup_set_closure(b))
    }
}

/// The subgroup set closure is unchanged after two applications.
theorem subgroup_set_closure_idempotent[G: Group](a: Set[G]) {
    subgroup_set_closure(subgroup_set_closure(a)) = subgroup_set_closure(a)
} by {
    subgroup_set_closure_at(a)
    subgroup_set_closure_at(subgroup_set_closure(a))
    subgroup_closure_idempotent(a)
    subgroup_set_closure(subgroup_set_closure(a)) = subgroup_set_closure(a)
}

/// Subgroup set closure is monotone as a set map.
theorem subgroup_set_closure_is_monotone[G: Group] {
    is_subset_monotone_map(subgroup_set_closure[G])
} by {
    forall(a: Set[G], b: Set[G]) {
        if a.subset(b) {
            subgroup_set_closure_mono(a, b)
            subgroup_set_closure(a).subset(subgroup_set_closure(b))
        }
    }
}

/// Subgroup set closure is extensive as a set map.
theorem subgroup_set_closure_is_extensive[G: Group] {
    is_set_extensive_map(subgroup_set_closure[G])
} by {
    forall(a: Set[G]) {
        subgroup_set_closure_extensive(a)
        a.subset(subgroup_set_closure(a))
    }
}

/// Subgroup set closure is idempotent as a set map.
theorem subgroup_set_closure_is_idempotent[G: Group] {
    is_set_idempotent_map(subgroup_set_closure[G])
} by {
    forall(a: Set[G]) {
        subgroup_set_closure_idempotent(a)
        subgroup_set_closure(subgroup_set_closure(a)) = subgroup_set_closure(a)
    }
}

/// Subgroup generation induces a closure operator on sets.
theorem subgroup_set_closure_operator[G: Group] {
    is_set_closure_operator(subgroup_set_closure[G])
} by {
    subgroup_set_closure_is_monotone[G]
    subgroup_set_closure_is_extensive[G]
    subgroup_set_closure_is_idempotent[G]
    is_set_closure_operator(subgroup_set_closure[G])
}

/// The closure of the empty set is the identity subgroup.
theorem subgroup_closure_empty[G: Group] {
    subgroup_closure(Set[G].empty_set) = identity_subgroup[G]
} by {
    set_subset_subgroup(Set[G].empty_set, identity_subgroup[G])
    subgroup_closure_subset_of_set_subset(Set[G].empty_set, identity_subgroup[G])
    subgroup_subset(subgroup_closure(Set[G].empty_set), identity_subgroup[G])
    identity_subgroup_subset(subgroup_closure(Set[G].empty_set))
    subgroup_subset_antisymm(subgroup_closure(Set[G].empty_set), identity_subgroup[G])
}

/// The closure of the universal set is the full subgroup.
theorem subgroup_closure_universal[G: Group] {
    subgroup_closure(Set[G].universal_set) = full_subgroup[G]
} by {
    subgroup_subset_full(subgroup_closure(Set[G].universal_set))
    forall(x: G) {
        if full_subgroup[G].contains(x) {
            Set[G].universal_set.contains(x)
            subgroup_closure_contains_of_set_contains(Set[G].universal_set, x)
            subgroup_closure(Set[G].universal_set).contains(x)
        }
    }
    subgroup_subset(full_subgroup[G], subgroup_closure(Set[G].universal_set))
    subgroup_subset_antisymm(subgroup_closure(Set[G].universal_set), full_subgroup[G])
}

/// The least subgroup containing two subgroups.
define subgroup_sup[G: Group](a: Subgroup[G], b: Subgroup[G]) -> Subgroup[G] {
    subgroup_closure(a.as_set.union(b.as_set))
}

/// The join of two subgroups is the closure of the union of their underlying sets.
theorem subgroup_sup_eq_closure_union[G: Group](a: Subgroup[G], b: Subgroup[G]) {
    subgroup_sup(a, b) = subgroup_closure(a.as_set.union(b.as_set))
}

/// The left subgroup is contained in the join.
theorem subgroup_subset_sup_left[G: Group](a: Subgroup[G], b: Subgroup[G]) {
    subgroup_subset(a, subgroup_sup(a, b))
} by {
    forall(x: G) {
        if a.contains(x) {
            subgroup_as_set_contains_eq(a, x)
            a.as_set.contains(x)
            union_contains_left(a.as_set, b.as_set, x)
            a.as_set.union(b.as_set).contains(x)
            subgroup_closure_contains_of_set_contains(a.as_set.union(b.as_set), x)
            subgroup_closure(a.as_set.union(b.as_set)).contains(x)
            subgroup_sup(a, b).contains(x)
        }
    }
}

/// The right subgroup is contained in the join.
theorem subgroup_subset_sup_right[G: Group](a: Subgroup[G], b: Subgroup[G]) {
    subgroup_subset(b, subgroup_sup(a, b))
} by {
    forall(x: G) {
        if b.contains(x) {
            subgroup_as_set_contains_eq(b, x)
            b.as_set.contains(x)
            union_contains_right(a.as_set, b.as_set, x)
            a.as_set.union(b.as_set).contains(x)
            subgroup_closure_contains_of_set_contains(a.as_set.union(b.as_set), x)
            subgroup_closure(a.as_set.union(b.as_set)).contains(x)
            subgroup_sup(a, b).contains(x)
        }
    }
}

/// The join is contained in every common upper bound.
theorem subgroup_sup_subset_of_subset_left_right[G: Group](
    a: Subgroup[G],
    b: Subgroup[G],
    c: Subgroup[G]
) {
    subgroup_subset(a, c) and subgroup_subset(b, c) implies subgroup_subset(subgroup_sup(a, b), c)
} by {
    if subgroup_subset(a, c) and subgroup_subset(b, c) {
        forall(x: G) {
            if a.as_set.union(b.as_set).contains(x) {
                union_contains_eq(a.as_set, b.as_set, x)
                if a.as_set.contains(x) {
                    subgroup_as_set_contains_eq(a, x)
                    a.contains(x)
                    c.contains(x)
                } else {
                    b.as_set.contains(x)
                    subgroup_as_set_contains_eq(b, x)
                    b.contains(x)
                    c.contains(x)
                }
            }
        }
        set_subset_subgroup(a.as_set.union(b.as_set), c)
        subgroup_closure_subset_of_set_subset(a.as_set.union(b.as_set), c)
        subgroup_subset(subgroup_closure(a.as_set.union(b.as_set)), c)
        subgroup_subset(subgroup_sup(a, b), c)
    }
}

/// Containment of a join is equivalent to containment of both subgroups.
theorem subgroup_sup_subset_iff[G: Group](
    a: Subgroup[G],
    b: Subgroup[G],
    c: Subgroup[G]
) {
    subgroup_subset(subgroup_sup(a, b), c) = (subgroup_subset(a, c) and subgroup_subset(b, c))
} by {
    if subgroup_subset(subgroup_sup(a, b), c) {
        subgroup_subset_sup_left(a, b)
        subgroup_subset_trans(a, subgroup_sup(a, b), c)
        subgroup_subset(a, c)
        subgroup_subset_sup_right(a, b)
        subgroup_subset_trans(b, subgroup_sup(a, b), c)
        subgroup_subset(b, c)
        subgroup_subset(a, c) and subgroup_subset(b, c)
    }
    if subgroup_subset(a, c) and subgroup_subset(b, c) {
        subgroup_sup_subset_of_subset_left_right(a, b, c)
        subgroup_subset(subgroup_sup(a, b), c)
    }
    subgroup_subset(subgroup_sup(a, b), c) = (subgroup_subset(a, c) and subgroup_subset(b, c))
}

attributes Subgroup[G: Group] {
    /// The smallest subgroup containing a set.
    let closure: Set[G] -> Subgroup[G] = subgroup_closure

    /// The least subgroup containing this subgroup and another subgroup.
    let sup: (Subgroup[G], Subgroup[G]) -> Subgroup[G] = subgroup_sup
}

/// True if a subgroup is generated by a finite set.
define subgroup_is_finitely_generated[G: Group](s: Subgroup[G]) -> Bool {
    exists(a: Set[G]) {
        a.is_finite and subgroup_closure(a) = s
    }
}

/// A finitely generated subgroup has a finite generating set.
theorem subgroup_is_finitely_generated_witness[G: Group](s: Subgroup[G]) {
    subgroup_is_finitely_generated(s) implies exists(a: Set[G]) {
        a.is_finite and subgroup_closure(a) = s
    }
}

/// A subgroup equal to the closure of a finite set is finitely generated.
theorem subgroup_is_finitely_generated_of_closure_eq[G: Group](a: Set[G], s: Subgroup[G]) {
    a.is_finite and subgroup_closure(a) = s implies subgroup_is_finitely_generated(s)
} by {
    if a.is_finite and subgroup_closure(a) = s {
        exists(b: Set[G]) {
            b.is_finite and subgroup_closure(b) = s
        }
        subgroup_is_finitely_generated(s)
    }
}

/// Equality preserves finite generation of subgroups.
theorem subgroup_is_finitely_generated_of_eq[G: Group](s: Subgroup[G], t: Subgroup[G]) {
    subgroup_is_finitely_generated(s) and s = t implies subgroup_is_finitely_generated(t)
} by {
    if subgroup_is_finitely_generated(s) and s = t {
        subgroup_is_finitely_generated_witness(s)
        let a: Set[G] satisfy {
            a.is_finite and subgroup_closure(a) = s
        }
        subgroup_closure(a) = t
        subgroup_is_finitely_generated_of_closure_eq(a, t)
        subgroup_is_finitely_generated(t)
    }
}

/// The closure of a finite set is a finitely generated subgroup.
theorem subgroup_closure_is_finitely_generated[G: Group](a: Set[G]) {
    a.is_finite implies subgroup_is_finitely_generated(subgroup_closure(a))
} by {
    if a.is_finite {
        subgroup_is_finitely_generated_of_closure_eq(a, subgroup_closure(a))
    }
}

/// The closure of one element is a finitely generated subgroup.
theorem subgroup_closure_singleton_is_finitely_generated[G: Group](a: G) {
    subgroup_is_finitely_generated(subgroup_closure(Set[G].singleton(a)))
} by {
    singleton_set_is_finite(a)
    subgroup_closure_is_finitely_generated(Set[G].singleton(a))
}

/// The generator belongs to the subgroup generated by it.
theorem subgroup_closure_singleton_contains[G: Group](a: G) {
    subgroup_closure(Set[G].singleton(a)).contains(a)
} by {
    singleton_contains_eq(a, a)
    subgroup_closure_contains_of_set_contains(Set[G].singleton(a), a)
}

/// The identity subgroup is finitely generated.
theorem identity_subgroup_is_finitely_generated[G: Group] {
    subgroup_is_finitely_generated(identity_subgroup[G])
} by {
    empty_set_is_finite[G]
    subgroup_closure_is_finitely_generated(Set[G].empty_set)
    subgroup_is_finitely_generated(subgroup_closure(Set[G].empty_set))
    subgroup_closure_empty[G]
    subgroup_is_finitely_generated(identity_subgroup[G])
}

/// The submonoid predicate associated to a subgroup.
theorem subgroup_submonoid_constraint[G: Group](s: Subgroup[G]) {
    submonoid_constraint(s.contains)
} by {
    subgroup_contains_identity(s)
    submonoid_identity_constraint(s.contains)
    forall(a: G, b: G) {
        if s.contains(a) and s.contains(b) {
            subgroup_mul_mem(s, a, b)
            s.contains(a * b)
        }
    }
    submonoid_closure_constraint(s.contains)
}

/// The submonoid associated to a subgroup.
let subgroup_to_submonoid[G: Group](s: Subgroup[G]) -> result: Submonoid[G] satisfy {
    Submonoid.new(s.contains) = Option.some(result)
} by {
    subgroup_submonoid_constraint(s)
}

/// The submonoid associated to a subgroup has the same membership predicate.
theorem subgroup_to_submonoid_contains[G: Group](s: Subgroup[G]) {
    subgroup_to_submonoid(s).contains = s.contains
}

/// Membership in the submonoid associated to a subgroup is membership in the subgroup.
theorem subgroup_to_submonoid_contains_eq[G: Group](s: Subgroup[G], x: G) {
    subgroup_to_submonoid(s).contains(x) = s.contains(x)
} by {
    subgroup_to_submonoid_contains(s)
    subgroup_to_submonoid(s).contains(x) = s.contains(x)
}

/// Membership in a subgroup is membership in its associated submonoid.
theorem subgroup_contains_to_submonoid_eq[G: Group](s: Subgroup[G], x: G) {
    s.contains(x) = subgroup_to_submonoid(s).contains(x)
} by {
    subgroup_to_submonoid_contains_eq(s, x)
    s.contains(x) = subgroup_to_submonoid(s).contains(x)
}

/// The associated submonoid has the same underlying set as the subgroup.
theorem subgroup_to_submonoid_as_set[G: Group](s: Subgroup[G]) {
    subgroup_to_submonoid(s).as_set = s.as_set
}

/// The inverse image membership predicate of a subgroup under a group homomorphism.
define subgroup_preimage_contains[G: Group, H: Group](
    f: GroupHom[G, H],
    s: Subgroup[H],
    a: G
) -> Bool {
    s.contains(f.hom(a))
}

/// The inverse image of a subgroup contains the identity.
theorem subgroup_preimage_identity_constraint[G: Group, H: Group](
    f: GroupHom[G, H],
    s: Subgroup[H]
) {
    identity_constraint(subgroup_preimage_contains(f, s))
} by {
    group_hom_one(f)
    f.hom(G.1) = H.1
    subgroup_contains_identity(s)
    s.contains(H.1)
    s.contains(f.hom(G.1))
    subgroup_preimage_contains(f, s, G.1)
}

/// The inverse image of a subgroup is closed under multiplication.
theorem subgroup_preimage_closure_constraint[G: Group, H: Group](
    f: GroupHom[G, H],
    s: Subgroup[H]
) {
    closure_constraint(subgroup_preimage_contains(f, s))
} by {
    forall(a: G, b: G) {
        if subgroup_preimage_contains(f, s, a) and subgroup_preimage_contains(f, s, b) {
            s.contains(f.hom(a))
            s.contains(f.hom(b))
            subgroup_mul_mem(s, f.hom(a), f.hom(b))
            s.contains(f.hom(a) * f.hom(b))
            group_hom_mul(f, a, b)
            f.hom(a * b) = f.hom(a) * f.hom(b)
            s.contains(f.hom(a * b))
            subgroup_preimage_contains(f, s, a * b)
        }
    }
}

/// The inverse image of a subgroup is closed under inverses.
theorem subgroup_preimage_inverse_constraint[G: Group, H: Group](
    f: GroupHom[G, H],
    s: Subgroup[H]
) {
    inverse_constraint(subgroup_preimage_contains(f, s))
} by {
    forall(a: G) {
        if subgroup_preimage_contains(f, s, a) {
            s.contains(f.hom(a))
            subgroup_inv_mem(s, f.hom(a))
            s.contains(f.hom(a).inverse)
            group_hom_inv(f, a)
            f.hom(a.inverse) = f.hom(a).inverse
            s.contains(f.hom(a.inverse))
            subgroup_preimage_contains(f, s, a.inverse)
        }
    }
}

/// The inverse image of a subgroup is a subgroup.
theorem subgroup_preimage_constraint[G: Group, H: Group](
    f: GroupHom[G, H],
    s: Subgroup[H]
) {
    subgroup_constraint(subgroup_preimage_contains(f, s))
} by {
    subgroup_preimage_identity_constraint(f, s)
    subgroup_preimage_closure_constraint(f, s)
    subgroup_preimage_inverse_constraint(f, s)
}

/// The inverse image of a subgroup under a group homomorphism.
let subgroup_preimage[G: Group, H: Group](f: GroupHom[G, H], s: Subgroup[H]) -> result: Subgroup[G] satisfy {
    Subgroup.new(subgroup_preimage_contains(f, s)) = Option.some(result)
} by {
    subgroup_preimage_constraint(f, s)
}

/// Membership in an inverse image means the mapped element belongs to the target subgroup.
theorem subgroup_preimage_contains_eq[G: Group, H: Group](
    f: GroupHom[G, H],
    s: Subgroup[H],
    a: G
) {
    subgroup_preimage(f, s).contains(a) = s.contains(f.hom(a))
} by {
    subgroup_preimage(f, s).contains(a) = subgroup_preimage_contains(f, s, a)
    subgroup_preimage_contains(f, s, a) = s.contains(f.hom(a))
}

/// Membership in a preimage follows from membership of the mapped element.
theorem subgroup_preimage_contains_of_contains_map[G: Group, H: Group](
    f: GroupHom[G, H],
    s: Subgroup[H],
    a: G
) {
    s.contains(f.hom(a)) implies subgroup_preimage(f, s).contains(a)
} by {
    if s.contains(f.hom(a)) {
        subgroup_preimage_contains_eq(f, s, a)
        subgroup_preimage(f, s).contains(a)
    }
}

/// Membership in the target follows from membership in the preimage.
theorem subgroup_contains_map_of_preimage_contains[G: Group, H: Group](
    f: GroupHom[G, H],
    s: Subgroup[H],
    a: G
) {
    subgroup_preimage(f, s).contains(a) implies s.contains(f.hom(a))
} by {
    if subgroup_preimage(f, s).contains(a) {
        subgroup_preimage_contains_eq(f, s, a)
        s.contains(f.hom(a))
    }
}

/// Inverse images are monotone with respect to subgroup containment.
theorem subgroup_preimage_subset_of_subset[G: Group, H: Group](
    f: GroupHom[G, H],
    s: Subgroup[H],
    t: Subgroup[H]
) {
    subgroup_subset(s, t) implies subgroup_subset(subgroup_preimage(f, s), subgroup_preimage(f, t))
} by {
    if subgroup_subset(s, t) {
        forall(a: G) {
            if subgroup_preimage(f, s).contains(a) {
                subgroup_preimage_contains_eq(f, s, a)
                s.contains(f.hom(a))
                subgroup_subset(s, t) = forall(x: H) {
                    s.contains(x) implies t.contains(x)
                }
                t.contains(f.hom(a))
                subgroup_preimage_contains_eq(f, t, a)
                subgroup_preimage(f, t).contains(a)
            }
        }
    }
}

/// The inverse image of an intersection is the intersection of the inverse images.
theorem subgroup_preimage_intersection[G: Group, H: Group](
    f: GroupHom[G, H],
    s: Subgroup[H],
    t: Subgroup[H]
) {
    subgroup_preimage(f, s.intersection(t)) =
        subgroup_preimage(f, s).intersection(subgroup_preimage(f, t))
} by {
    forall(a: G) {
        subgroup_preimage_contains_eq(f, s.intersection(t), a)
        subgroup_preimage_contains_eq(f, s, a)
        subgroup_preimage_contains_eq(f, t, a)
        subgroup_intersection_contains_eq(s, t, f.hom(a))
        subgroup_intersection_contains_eq(subgroup_preimage(f, s), subgroup_preimage(f, t), a)
        subgroup_preimage(f, s.intersection(t)).contains(a) =
            subgroup_preimage(f, s).intersection(subgroup_preimage(f, t)).contains(a)
    }
    subgroup_ext(subgroup_preimage(f, s.intersection(t)),
        subgroup_preimage(f, s).intersection(subgroup_preimage(f, t)))
}

/// The inverse image of the full subgroup is the full subgroup.
theorem subgroup_preimage_full[G: Group, H: Group](f: GroupHom[G, H]) {
    subgroup_preimage(f, full_subgroup[H]) = full_subgroup[G]
} by {
    subgroup_subset_full(subgroup_preimage(f, full_subgroup[H]))
    forall(a: G) {
        if full_subgroup[G].contains(a) {
            full_subgroup_contains_everything(f.hom(a))
            subgroup_preimage_contains_eq(f, full_subgroup[H], a)
            subgroup_preimage(f, full_subgroup[H]).contains(a)
        }
    }
    subgroup_subset(full_subgroup[G], subgroup_preimage(f, full_subgroup[H]))
    subgroup_subset_antisymm(subgroup_preimage(f, full_subgroup[H]), full_subgroup[G])
}

/// The kernel of a group homomorphism as a subgroup.
define group_hom_kernel[G: Group, H: Group](f: GroupHom[G, H]) -> Subgroup[G] {
    subgroup_preimage(f, identity_subgroup[H])
}

/// Membership in the kernel means the homomorphism maps the element to the identity.
theorem group_hom_kernel_contains_eq[G: Group, H: Group](f: GroupHom[G, H], a: G) {
    group_hom_kernel(f).contains(a) = (f.hom(a) = H.1)
} by {
    group_hom_kernel(f) = subgroup_preimage(f, identity_subgroup[H])
    subgroup_preimage_contains_eq(f, identity_subgroup[H], a)
    if group_hom_kernel(f).contains(a) {
        identity_subgroup[H].contains(f.hom(a))
        identity_subgroup_only_has_identity(f.hom(a))
        f.hom(a) = H.1
    }
    if f.hom(a) = H.1 {
        subgroup_contains_identity(identity_subgroup[H])
        identity_subgroup[H].contains(H.1)
        identity_subgroup[H].contains(f.hom(a))
        group_hom_kernel(f).contains(a)
    }
    group_hom_kernel(f).contains(a) = (f.hom(a) = H.1)
}

/// An element belongs to the kernel when its image is the identity.
theorem group_hom_kernel_contains_of_maps_to_identity[G: Group, H: Group](f: GroupHom[G, H], a: G) {
    f.hom(a) = H.1 implies group_hom_kernel(f).contains(a)
} by {
    if f.hom(a) = H.1 {
        group_hom_kernel_contains_eq(f, a)
        group_hom_kernel(f).contains(a)
    }
}

/// A kernel element maps to the identity.
theorem group_hom_maps_to_identity_of_kernel_contains[G: Group, H: Group](f: GroupHom[G, H], a: G) {
    group_hom_kernel(f).contains(a) implies f.hom(a) = H.1
} by {
    if group_hom_kernel(f).contains(a) {
        group_hom_kernel_contains_eq(f, a)
        f.hom(a) = H.1
    }
}


/// Subgroup join is commutative.
theorem subgroup_sup_comm[G: Group](a: Subgroup[G], b: Subgroup[G]) {
    subgroup_sup(a, b) = subgroup_sup(b, a)
} by {
    subgroup_subset_sup_right(b, a)
    subgroup_subset_sup_left(b, a)
    subgroup_sup_subset_of_subset_left_right(a, b, subgroup_sup(b, a))
    subgroup_subset_sup_right(a, b)
    subgroup_subset_sup_left(a, b)
    subgroup_sup_subset_of_subset_left_right(b, a, subgroup_sup(a, b))
    subgroup_subset_antisymm(subgroup_sup(a, b), subgroup_sup(b, a))
}

/// Joining a subgroup with itself gives the same subgroup.
theorem subgroup_sup_idempotent[G: Group](a: Subgroup[G]) {
    subgroup_sup(a, a) = a
} by {
    subgroup_subset_refl(a)
    subgroup_sup_subset_of_subset_left_right(a, a, a)
    subgroup_subset_sup_left(a, a)
    subgroup_subset_antisymm(subgroup_sup(a, a), a)
}

/// Subgroup join is associative.
theorem subgroup_sup_assoc[G: Group](
    a: Subgroup[G],
    b: Subgroup[G],
    c: Subgroup[G]
) {
    subgroup_sup(subgroup_sup(a, b), c) = subgroup_sup(a, subgroup_sup(b, c))
} by {
    subgroup_subset_sup_left(a, subgroup_sup(b, c))
    subgroup_subset_sup_right(a, subgroup_sup(b, c))
    subgroup_subset_sup_left(b, c)
    subgroup_subset_sup_right(b, c)
    subgroup_subset_trans(b, subgroup_sup(b, c), subgroup_sup(a, subgroup_sup(b, c)))
    subgroup_subset_trans(c, subgroup_sup(b, c), subgroup_sup(a, subgroup_sup(b, c)))
    subgroup_sup_subset_of_subset_left_right(a, b, subgroup_sup(a, subgroup_sup(b, c)))
    subgroup_sup_subset_of_subset_left_right(subgroup_sup(a, b), c, subgroup_sup(a, subgroup_sup(b, c)))
    subgroup_subset_sup_left(subgroup_sup(a, b), c)
    subgroup_subset_sup_right(subgroup_sup(a, b), c)
    subgroup_subset_sup_left(a, b)
    subgroup_subset_sup_right(a, b)
    subgroup_subset_trans(a, subgroup_sup(a, b), subgroup_sup(subgroup_sup(a, b), c))
    subgroup_subset_trans(b, subgroup_sup(a, b), subgroup_sup(subgroup_sup(a, b), c))
    subgroup_sup_subset_of_subset_left_right(b, c, subgroup_sup(subgroup_sup(a, b), c))
    subgroup_sup_subset_of_subset_left_right(a, subgroup_sup(b, c), subgroup_sup(subgroup_sup(a, b), c))
    subgroup_subset_antisymm(subgroup_sup(subgroup_sup(a, b), c), subgroup_sup(a, subgroup_sup(b, c)))
}
