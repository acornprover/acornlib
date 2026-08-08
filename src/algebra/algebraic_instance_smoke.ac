from nat import Nat, semiring_zero_pow
from int import Int
from rat import Rat
from real import Real
from algebra.ring.ring import mul_neg_one_left
from algebra.field.field import field_mul_eq_zero, pow_not_zero
from algebra.integral_domain import integral_domain_pow_nonzero
from algebra.comm_ring_unit import field_is_unit_iff_nonzero, field_nonzero_pow_is_unit
from ordered_field import multiply_inequality_with_nonnegative_element,
    pows_of_nonnegative_are_nonnegative, positive_pows_of_positive_are_positive,
    inverse_of_nonnegative_is_nonnegative, inverse_of_positive_is_positive

/// The natural numbers expose the semiring zero-power theorem.
theorem nat_semiring_zero_pow_smoke(n: Nat) {
    n != Nat.0 implies Nat.0.pow(n) = Nat.0
} by {
    semiring_zero_pow[Nat](n)
}

/// The integers expose inherited ring multiplication by negative one.
theorem int_ring_neg_one_mul_smoke(a: Int) {
    -Int.1 * a = -a
} by {
    mul_neg_one_left[Int](a)
}

/// The rationals expose the field zero-product bridge.
theorem rat_field_mul_eq_zero_smoke(a: Rat, b: Rat) {
    a * b = Rat.0 implies a = Rat.0 or b = Rat.0
} by {
    field_mul_eq_zero(a, b)
}

/// The rationals expose the field unit/nonzero equivalence.
theorem rat_field_unit_nonzero_smoke(a: Rat) {
    a.is_unit = (a != Rat.0)
} by {
    field_is_unit_iff_nonzero(a)
}

/// The rationals expose ordered-field multiplication by a nonnegative scalar.
theorem rat_ordered_field_nonnegative_mul_smoke(a: Rat, b: Rat) {
    Rat.0 <= a and Rat.0 <= b implies Rat.0 <= a * b
} by {
    if Rat.0 <= a and Rat.0 <= b {
        multiply_inequality_with_nonnegative_element(Rat.0, a, b)
        Rat.0 <= a * b
    }
}

/// The rationals expose ordered-field nonnegative powers.
theorem rat_ordered_field_nonnegative_pow_smoke(a: Rat, n: Nat) {
    Rat.0 <= a implies Rat.0 <= a.pow(n)
} by {
    pows_of_nonnegative_are_nonnegative(a, n)
}

/// The rationals expose ordered-field positive powers.
theorem rat_ordered_field_positive_pow_smoke(a: Rat, n: Nat) {
    Rat.0 < a implies Rat.0 < a.pow(n)
} by {
    positive_pows_of_positive_are_positive(a, n)
}

/// The rationals expose ordered-field nonnegative inverses.
theorem rat_ordered_field_inverse_nonnegative_smoke(a: Rat) {
    Rat.0 <= a implies Rat.0 <= a.inverse
} by {
    inverse_of_nonnegative_is_nonnegative(a)
}

/// The rationals expose ordered-field positive inverses.
theorem rat_ordered_field_inverse_positive_smoke(a: Rat) {
    Rat.0 < a implies Rat.0 < a.inverse
} by {
    inverse_of_positive_is_positive(a)
}

/// The reals expose the field zero-product bridge.
theorem real_field_mul_eq_zero_smoke(a: Real, b: Real) {
    a * b = Real.0 implies a = Real.0 or b = Real.0
} by {
    field_mul_eq_zero(a, b)
}

/// The reals expose the field nonzero-power theorem.
theorem real_field_pow_not_zero_smoke(a: Real, n: Nat) {
    a != Real.0 implies a.pow(n) != Real.0
} by {
    pow_not_zero(a, n)
}

/// The reals expose the integral-domain nonzero-power theorem.
theorem real_integral_domain_pow_nonzero_smoke(a: Real, n: Nat) {
    a != Real.0 implies a.pow(n) != Real.0
} by {
    integral_domain_pow_nonzero(a, n)
}

/// The reals expose the field nonzero-power unit bridge.
theorem real_field_nonzero_pow_is_unit_smoke(a: Real, n: Nat) {
    a != Real.0 implies a.pow(n).is_unit
} by {
    field_nonzero_pow_is_unit(a, n)
}

/// The reals expose ordered-field multiplication by a nonnegative scalar.
theorem real_ordered_field_nonnegative_mul_smoke(a: Real, b: Real) {
    Real.0 <= a and Real.0 <= b implies Real.0 <= a * b
} by {
    if Real.0 <= a and Real.0 <= b {
        multiply_inequality_with_nonnegative_element(Real.0, a, b)
        Real.0 <= a * b
    }
}

/// The reals expose ordered-field nonnegative powers.
theorem real_ordered_field_nonnegative_pow_smoke(a: Real, n: Nat) {
    Real.0 <= a implies Real.0 <= a.pow(n)
} by {
    pows_of_nonnegative_are_nonnegative(a, n)
}

/// The reals expose ordered-field positive powers.
theorem real_ordered_field_positive_pow_smoke(a: Real, n: Nat) {
    Real.0 < a implies Real.0 < a.pow(n)
} by {
    positive_pows_of_positive_are_positive(a, n)
}

/// The reals expose ordered-field nonnegative inverses.
theorem real_ordered_field_inverse_nonnegative_smoke(a: Real) {
    Real.0 <= a implies Real.0 <= a.inverse
} by {
    inverse_of_nonnegative_is_nonnegative(a)
}

/// The reals expose ordered-field positive inverses.
theorem real_ordered_field_inverse_positive_smoke(a: Real) {
    Real.0 < a implies Real.0 < a.inverse
} by {
    inverse_of_positive_is_positive(a)
}
