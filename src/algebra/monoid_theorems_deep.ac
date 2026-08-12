// Monoid deepening: power laws in a general monoid (successor powers, powers
// of the same element commuting, commuting with the powers of a commuting
// partner), the only idempotent element of a group being the identity, and
// the distribution of powers over products of commuting elements.

from algebra.monoid.monoid import Monoid
from algebra.group import Group, left_cancel
from algebra.idempotent_elem import is_idempotent_elem
from algebra.comm_monoid import CommMonoid
from nat import Nat, pow_one, pow_add, pow_zero, pow_pow, alt_induction, add_comm, mul_comm,
    pow_distrib_mul
numerals Nat

/// The successor power of an element is the product with itself on the left.
theorem monoid_pow_suc_left[M: Monoid](a: M, n: Nat) {
    a.pow(n.suc) = a * a.pow(n)
}

/// The successor power of an element is the product with itself on the right.
theorem monoid_pow_suc_right[M: Monoid](a: M, n: Nat) {
    a.pow(n.suc) = a.pow(n) * a
} by {
    pow_add(a, n, Nat.1)
    a.pow(n) * a.pow(Nat.1) = a.pow(n + Nat.1)
    pow_one(a)
    a.pow(Nat.1) = a
    n + Nat.1 = n.suc
    a.pow(n + Nat.1) = a.pow(n.suc)
    a.pow(n) * a = a.pow(n.suc)
}

/// Powers of the same element commute: a^m * a^n = a^n * a^m.
theorem monoid_pow_comm[M: Monoid](a: M, m: Nat, n: Nat) {
    a.pow(m) * a.pow(n) = a.pow(n) * a.pow(m)
} by {
    pow_add(a, m, n)
    a.pow(m) * a.pow(n) = a.pow(m + n)
    pow_add(a, n, m)
    a.pow(n) * a.pow(m) = a.pow(n + m)
    add_comm(m, n)
    m + n = n + m
    a.pow(m + n) = a.pow(n + m)
    a.pow(m) * a.pow(n) = a.pow(n) * a.pow(m)
}

/// An element commutes with its own powers: a * a^n = a^n * a.
theorem monoid_pow_comm_self[M: Monoid](a: M, n: Nat) {
    a * a.pow(n) = a.pow(n) * a
} by {
    monoid_pow_suc_left(a, n)
    a.pow(n.suc) = a * a.pow(n)
    monoid_pow_suc_right(a, n)
    a.pow(n.suc) = a.pow(n) * a
    a * a.pow(n) = a.pow(n) * a
}

/// Powers of a power multiply exponents in either order.
theorem monoid_pow_pow_comm[M: Monoid](a: M, m: Nat, n: Nat) {
    a.pow(m).pow(n) = a.pow(n).pow(m)
} by {
    pow_pow(a, m, n)
    a.pow(m).pow(n) = a.pow(m * n)
    pow_pow(a, n, m)
    a.pow(n).pow(m) = a.pow(n * m)
    mul_comm(m, n)
    m * n = n * m
    a.pow(m * n) = a.pow(n * m)
    a.pow(m).pow(n) = a.pow(n).pow(m)
}

/// The only idempotent element of a group is the identity.
theorem group_idempotent_is_identity[G: Group](a: G) {
    is_idempotent_elem(a) implies a = G.1
} by {
    if is_idempotent_elem(a) {
        is_idempotent_elem(a) = (a * a = a)
        a * a = a
        a * a = a * G.1
        left_cancel(a, a, G.1)
        a = G.1
    }
}

/// A commutative monoid power distributes over products.
theorem comm_monoid_pow_mul[M: CommMonoid](a: M, b: M, n: Nat) {
    (a * b).pow(n) = a.pow(n) * b.pow(n)
} by {
    pow_distrib_mul(a, b, n)
    (a * b).pow(n) = a.pow(n) * b.pow(n)
}

/// An element commutes with every power of an element it commutes with.
theorem monoid_comm_pow_right[M: Monoid](a: M, b: M, n: Nat) {
    a * b = b * a implies a * b.pow(n) = b.pow(n) * a
} by {
    define p(k: Nat) -> Bool {
        a * b = b * a implies a * b.pow(k) = b.pow(k) * a
    }
    pow_zero(b)
    b.pow(Nat.0) = M.1
    a * b.pow(Nat.0) = a * M.1
    a * M.1 = a
    b.pow(Nat.0) * a = M.1 * a
    M.1 * a = a
    a * b.pow(Nat.0) = b.pow(Nat.0) * a
    if a * b = b * a {
        a * b.pow(Nat.0) = b.pow(Nat.0) * a
    }
    p(Nat.0)
    forall(k: Nat) {
        if p(k) {
            if a * b = b * a {
                pow_add(b, k, Nat.1)
                b.pow(k) * b.pow(Nat.1) = b.pow(k + Nat.1)
                pow_one(b)
                b.pow(Nat.1) = b
                k + Nat.1 = k.suc
                b.pow(k + Nat.1) = b.pow(k.suc)
                b.pow(k) * b = b.pow(k.suc)
                a * b.pow(k.suc) = a * (b.pow(k) * b)
                a * (b.pow(k) * b) = (a * b.pow(k)) * b
                a * b.pow(k) = b.pow(k) * a
                (a * b.pow(k)) * b = (b.pow(k) * a) * b
                (b.pow(k) * a) * b = b.pow(k) * (a * b)
                a * b = b * a
                b.pow(k) * (a * b) = b.pow(k) * (b * a)
                b.pow(k) * (b * a) = (b.pow(k) * b) * a
                b.pow(k) * b = b.pow(k.suc)
                (b.pow(k) * b) * a = b.pow(k.suc) * a
                a * b.pow(k.suc) = b.pow(k.suc) * a
            }
            p(k.suc)
        }
    }
    p(Nat.0) and forall(k: Nat) { p(k) implies p(k.suc) }
    alt_induction(p)
    forall(k: Nat) { p(k) }
}

/// The inductive step of the commuting-power law.
theorem monoid_comm_pow_add_step[M: Monoid](a: M, b: M, k: Nat) {
    a * b = b * a and (a * b).pow(k) = a.pow(k) * b.pow(k)
        implies (a * b).pow(k.suc) = a.pow(k.suc) * b.pow(k.suc)
} by {
    if a * b = b * a and (a * b).pow(k) = a.pow(k) * b.pow(k) {
        monoid_pow_suc_left(a * b, k)
        (a * b).pow(k.suc) = (a * b) * (a * b).pow(k)
        (a * b).pow(k) = a.pow(k) * b.pow(k)
        (a * b) * (a * b).pow(k) = (a * b) * (a.pow(k) * b.pow(k))
        (a * b) * (a.pow(k) * b.pow(k)) = a * (b * (a.pow(k) * b.pow(k)))
        b * (a.pow(k) * b.pow(k)) = (b * a.pow(k)) * b.pow(k)
        monoid_comm_pow_right(b, a, k)
        b * a = a * b
        b * a.pow(k) = a.pow(k) * b
        (b * a.pow(k)) * b.pow(k) = (a.pow(k) * b) * b.pow(k)
        (a.pow(k) * b) * b.pow(k) = a.pow(k) * (b * b.pow(k))
        b * b.pow(k) = b.pow(k.suc)
        a.pow(k) * (b * b.pow(k)) = a.pow(k) * b.pow(k.suc)
        a * (a.pow(k) * b.pow(k.suc)) = (a * a.pow(k)) * b.pow(k.suc)
        a * a.pow(k) = a.pow(k.suc)
        (a * a.pow(k)) * b.pow(k.suc) = a.pow(k.suc) * b.pow(k.suc)
        (a * b).pow(k.suc) = a.pow(k.suc) * b.pow(k.suc)
    }
}

/// The power of a product of commuting elements is the product of the powers.
theorem monoid_comm_pow_add[M: Monoid](a: M, b: M, n: Nat) {
    a * b = b * a implies (a * b).pow(n) = a.pow(n) * b.pow(n)
} by {
    define p(k: Nat) -> Bool {
        a * b = b * a implies (a * b).pow(k) = a.pow(k) * b.pow(k)
    }
    pow_zero(a * b)
    (a * b).pow(Nat.0) = M.1
    pow_zero(a)
    a.pow(Nat.0) = M.1
    pow_zero(b)
    b.pow(Nat.0) = M.1
    a.pow(Nat.0) * b.pow(Nat.0) = M.1 * M.1
    M.1 * M.1 = M.1
    (a * b).pow(Nat.0) = a.pow(Nat.0) * b.pow(Nat.0)
    if a * b = b * a {
        (a * b).pow(Nat.0) = a.pow(Nat.0) * b.pow(Nat.0)
    }
    p(Nat.0)
    forall(k: Nat) {
        if p(k) {
            if a * b = b * a {
                p(k) = (a * b = b * a implies (a * b).pow(k) = a.pow(k) * b.pow(k))
                (a * b).pow(k) = a.pow(k) * b.pow(k)
                monoid_comm_pow_add_step(a, b, k)
                (a * b).pow(k.suc) = a.pow(k.suc) * b.pow(k.suc)
            }
            p(k.suc)
        }
    }
    p(Nat.0) and forall(k: Nat) { p(k) implies p(k.suc) }
    alt_induction(p)
    forall(k: Nat) { p(k) }
}
