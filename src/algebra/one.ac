/// The "One" typeclass represents anything that has a multiplicative identity element.
typeclass A: One {
    1: A
}
