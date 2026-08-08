from algebra.add_monoid import AddMonoid, AddMonoidHom, add_monoid_hom_zero, add_monoid_hom_add
from data.basic.set import Set, is_subset_monotone_map, is_set_extensive_map, is_set_idempotent_map,
    is_set_closure_operator, union_contains_eq, union_contains_left, union_contains_right
from data.basic.functions import predicate_extensionality

/// True if a subset contains the additive identity.
define add_submonoid_zero_constraint[M: AddMonoid](contains: M -> Bool) -> Bool {
    contains(M.0)
}

/// True if a subset is closed under addition.
define add_submonoid_closure_constraint[M: AddMonoid](contains: M -> Bool) -> Bool {
    forall(a: M, b: M) {
        contains(a) and contains(b) implies contains(a + b)
    }
}

/// True if a subset satisfies all the requirements to be an additive submonoid.
define add_submonoid_constraint[M: AddMonoid](contains: M -> Bool) -> Bool {
    add_submonoid_zero_constraint(contains) and add_submonoid_closure_constraint(contains)
}

/// An additive submonoid of an additive monoid, represented as a subset containing zero and closed under addition.
structure AddSubmonoid[M: AddMonoid] {
    /// True if the given element is a member of this additive submonoid.
    contains: M -> Bool
} constraint {
    add_submonoid_constraint(contains)
}

/// Additive submonoid extensionality from equality of membership.
theorem add_submonoid_ext[M: AddMonoid](a: AddSubmonoid[M], b: AddSubmonoid[M]) {
    (forall(x: M) { a.contains(x) = b.contains(x) }) implies a = b
} by {
    if forall(x: M) { a.contains(x) = b.contains(x) } {
        a.contains = b.contains
    }
}

/// Equal additive submonoids have equal membership predicates.
theorem add_submonoid_eq_contains[M: AddMonoid](a: AddSubmonoid[M], b: AddSubmonoid[M]) {
    a = b implies a.contains = b.contains
}

/// Equal additive submonoids have equal membership at every element.
theorem add_submonoid_eq_contains_at[M: AddMonoid](a: AddSubmonoid[M], b: AddSubmonoid[M], x: M) {
    a = b implies a.contains(x) = b.contains(x)
} by {
    if a = b {
        a.contains(x) = b.contains(x)
    }
}

/// Equality of additive submonoids transports predicates on additive submonoids.
theorem add_submonoid_eq_transport_predicate[M: AddMonoid](
    p: AddSubmonoid[M] -> Bool,
    a: AddSubmonoid[M],
    b: AddSubmonoid[M]
) {
    a = b and p(a) implies p(b)
}

/// Equality of additive submonoids transports predicates on additive submonoids in the reverse direction.
theorem add_submonoid_eq_transport_predicate_rev[M: AddMonoid](
    p: AddSubmonoid[M] -> Bool,
    a: AddSubmonoid[M],
    b: AddSubmonoid[M]
) {
    a = b and p(b) implies p(a)
}

/// Equality of membership predicates determines equality of additive submonoids.
theorem add_submonoid_eq_of_contains_eq[M: AddMonoid](a: AddSubmonoid[M], b: AddSubmonoid[M]) {
    a.contains = b.contains implies a = b
} by {
    if a.contains = b.contains {
        forall(x: M) {
            a.contains(x) = b.contains(x)
        }
        add_submonoid_ext(a, b)
    }
}

/// Pointwise equality of membership determines equality of additive submonoids.
theorem add_submonoid_eq_of_contains_at_eq[M: AddMonoid](a: AddSubmonoid[M], b: AddSubmonoid[M]) {
    (forall(x: M) { a.contains(x) = b.contains(x) }) implies a = b
} by {
    if forall(x: M) { a.contains(x) = b.contains(x) } {
        add_submonoid_ext(a, b)
    }
}

/// Membership of zero in any additive submonoid.
theorem add_submonoid_contains_zero[M: AddMonoid](s: AddSubmonoid[M]) {
    s.contains(M.0)
} by {
    add_submonoid_zero_constraint(s.contains)
}

/// Closure of any additive submonoid under addition.
theorem add_submonoid_contains_add[M: AddMonoid](s: AddSubmonoid[M], a: M, b: M) {
    s.contains(a) and s.contains(b) implies s.contains(a + b)
} by {
    if s.contains(a) and s.contains(b) {
        add_submonoid_closure_constraint(s.contains)
        add_submonoid_closure_constraint(s.contains) = forall(x: M, y: M) {
            s.contains(x) and s.contains(y) implies s.contains(x + y)
        }
        s.contains(a + b)
    }
}

/// The common membership predicate of two additive submonoids.
define add_submonoid_intersection_contains[M: AddMonoid](
    a: AddSubmonoid[M],
    b: AddSubmonoid[M],
    x: M
) -> Bool {
    a.contains(x) and b.contains(x)
}

/// The common part of two additive submonoids contains zero.
theorem add_submonoid_intersection_zero_constraint[M: AddMonoid](a: AddSubmonoid[M], b: AddSubmonoid[M]) {
    add_submonoid_zero_constraint(add_submonoid_intersection_contains(a, b))
} by {
    add_submonoid_contains_zero(a)
    add_submonoid_contains_zero(b)
}

/// The common part of two additive submonoids is closed under addition.
theorem add_submonoid_intersection_closure_constraint[M: AddMonoid](a: AddSubmonoid[M], b: AddSubmonoid[M]) {
    add_submonoid_closure_constraint(add_submonoid_intersection_contains(a, b))
} by {
    forall(x: M, y: M) {
        if add_submonoid_intersection_contains(a, b, x) and add_submonoid_intersection_contains(a, b, y) {
            a.contains(y)
            b.contains(y)
            add_submonoid_contains_add(a, x, y)
            add_submonoid_contains_add(b, x, y)
            b.contains(x + y)
            add_submonoid_intersection_contains(a, b, x + y)
        }
    }
}

/// The common part of two additive submonoids is an additive submonoid.
theorem add_submonoid_intersection_constraint[M: AddMonoid](a: AddSubmonoid[M], b: AddSubmonoid[M]) {
    add_submonoid_constraint(add_submonoid_intersection_contains(a, b))
} by {
    add_submonoid_intersection_zero_constraint(a, b)
    add_submonoid_intersection_closure_constraint(a, b)
}

/// The common part of two additive submonoids.
let add_submonoid_intersection[M: AddMonoid](a: AddSubmonoid[M], b: AddSubmonoid[M]) -> result: AddSubmonoid[M] satisfy {
    AddSubmonoid.new(add_submonoid_intersection_contains(a, b)) = Option.some(result)
} by {
    add_submonoid_intersection_constraint(a, b)
}

attributes AddSubmonoid[M: AddMonoid] {
    /// Additive submonoid extensionality from pointwise equality of membership.
    let ext = add_submonoid_ext[M]

    /// The subset of additive monoid elements belonging to this additive submonoid.
    define as_set(self) -> Set[M] {
        Set[M].new(self.contains)
    }

    /// The common part of two additive submonoids.
    let intersection: (AddSubmonoid[M], AddSubmonoid[M]) -> AddSubmonoid[M] = add_submonoid_intersection
}

/// Membership in the underlying set is membership in the additive submonoid.
theorem add_submonoid_as_set_contains_eq[M: AddMonoid](s: AddSubmonoid[M], x: M) {
    s.as_set.contains(x) = s.contains(x)
} by {
}

/// Membership in an additive submonoid is membership in its underlying set.
theorem add_submonoid_contains_as_set_eq[M: AddMonoid](s: AddSubmonoid[M], x: M) {
    s.contains(x) = s.as_set.contains(x)
} by {
    add_submonoid_as_set_contains_eq(s, x)
}

/// Equal additive submonoids have equal underlying sets.
theorem add_submonoid_eq_as_set[M: AddMonoid](a: AddSubmonoid[M], b: AddSubmonoid[M]) {
    a = b implies a.as_set = b.as_set
}

/// Membership in the intersection means membership in both additive submonoids.
theorem add_submonoid_intersection_contains_eq[M: AddMonoid](a: AddSubmonoid[M], b: AddSubmonoid[M],
    x: M) {
    a.intersection(b).contains(x) = (a.contains(x) and b.contains(x))
} by {
    a.intersection(b).contains(x) = add_submonoid_intersection_contains(a, b, x)
}

/// The intersection is contained in the left additive submonoid.
theorem add_submonoid_intersection_subset_left[M: AddMonoid](a: AddSubmonoid[M], b: AddSubmonoid[M],
    x: M) {
    a.intersection(b).contains(x) implies a.contains(x)
} by {
    if a.intersection(b).contains(x) {
        add_submonoid_intersection_contains_eq(a, b, x)
        a.contains(x)
    }
}

/// The intersection is contained in the right additive submonoid.
theorem add_submonoid_intersection_subset_right[M: AddMonoid](a: AddSubmonoid[M], b: AddSubmonoid[M],
    x: M) {
    a.intersection(b).contains(x) implies b.contains(x)
} by {
    if a.intersection(b).contains(x) {
        add_submonoid_intersection_contains_eq(a, b, x)
        b.contains(x)
    }
}

/// Additive submonoid intersection is commutative.
theorem add_submonoid_intersection_comm[M: AddMonoid](a: AddSubmonoid[M], b: AddSubmonoid[M]) {
    a.intersection(b) = b.intersection(a)
} by {
    forall(x: M) {
        if a.intersection(b).contains(x) {
            add_submonoid_intersection_contains_eq(a, b, x)
            add_submonoid_intersection_contains_eq(b, a, x)
            b.intersection(a).contains(x)
        }
        if b.intersection(a).contains(x) {
            add_submonoid_intersection_contains_eq(b, a, x)
            add_submonoid_intersection_contains_eq(a, b, x)
            a.intersection(b).contains(x)
        }
        a.intersection(b).contains(x) = b.intersection(a).contains(x)
    }
    add_submonoid_ext(a.intersection(b), b.intersection(a))
}

/// Additive submonoid intersection is associative.
theorem add_submonoid_intersection_assoc[M: AddMonoid](
    a: AddSubmonoid[M],
    b: AddSubmonoid[M],
    c: AddSubmonoid[M]
) {
    a.intersection(b).intersection(c) = a.intersection(b.intersection(c))
} by {
    let lhs = a.intersection(b).intersection(c)
    let rhs = a.intersection(b.intersection(c))
    forall(x: M) {
        if lhs.contains(x) {
            add_submonoid_intersection_contains_eq(a.intersection(b), c, x)
            add_submonoid_intersection_contains_eq(a, b, x)
            b.contains(x)
            add_submonoid_intersection_contains_eq(b, c, x)
            b.intersection(c).contains(x)
            add_submonoid_intersection_contains_eq(a, b.intersection(c), x)
            rhs.contains(x)
        }
        if rhs.contains(x) {
            add_submonoid_intersection_contains_eq(a, b.intersection(c), x)
            add_submonoid_intersection_contains_eq(b, c, x)
            b.contains(x)
            add_submonoid_intersection_contains_eq(a, b, x)
            a.intersection(b).contains(x)
            add_submonoid_intersection_contains_eq(a.intersection(b), c, x)
            lhs.contains(x)
        }
        lhs.contains(x) = rhs.contains(x)
    }
    add_submonoid_ext(lhs, rhs)
}

/// Additive submonoid intersection is idempotent.
theorem add_submonoid_intersection_idempotent[M: AddMonoid](a: AddSubmonoid[M]) {
    a.intersection(a) = a
} by {
    forall(x: M) {
        if a.intersection(a).contains(x) {
            add_submonoid_intersection_contains_eq(a, a, x)
            a.contains(x)
        }
        if a.contains(x) {
            add_submonoid_intersection_contains_eq(a, a, x)
            a.intersection(a).contains(x)
        }
        a.intersection(a).contains(x) = a.contains(x)
    }
    add_submonoid_ext(a.intersection(a), a)
}

/// True if every element of one additive submonoid belongs to another.
define add_submonoid_subset[M: AddMonoid](a: AddSubmonoid[M], b: AddSubmonoid[M]) -> Bool {
    forall(x: M) {
        a.contains(x) implies b.contains(x)
    }
}

/// Additive submonoid containment is containment of the underlying sets.
theorem add_submonoid_subset_as_set_eq[M: AddMonoid](a: AddSubmonoid[M], b: AddSubmonoid[M]) {
    add_submonoid_subset(a, b) = a.as_set.subset(b.as_set)
} by {
    if add_submonoid_subset(a, b) {
        forall(x: M) {
            if a.as_set.contains(x) {
                add_submonoid_as_set_contains_eq(a, x)
                a.contains(x)
                b.contains(x)
                add_submonoid_as_set_contains_eq(b, x)
                b.as_set.contains(x)
            }
        }
        a.as_set.subset(b.as_set)
    }
    if a.as_set.subset(b.as_set) {
        forall(x: M) {
            if a.contains(x) {
                add_submonoid_as_set_contains_eq(a, x)
                a.as_set.subset(b.as_set) = forall(y: M) {
                    a.as_set.contains(y) implies b.as_set.contains(y)
                }
                add_submonoid_as_set_contains_eq(b, x)
                b.contains(x)
            }
        }
        add_submonoid_subset(a, b)
    }
}

/// Additive submonoid containment gives containment of underlying sets.
theorem add_submonoid_as_set_subset_of_subset[M: AddMonoid](a: AddSubmonoid[M], b: AddSubmonoid[M]) {
    add_submonoid_subset(a, b) implies a.as_set.subset(b.as_set)
} by {
    if add_submonoid_subset(a, b) {
        add_submonoid_subset_as_set_eq(a, b)
        a.as_set.subset(b.as_set)
    }
}

/// Containment of underlying sets gives additive submonoid containment.
theorem add_submonoid_subset_of_as_set_subset[M: AddMonoid](a: AddSubmonoid[M], b: AddSubmonoid[M]) {
    a.as_set.subset(b.as_set) implies add_submonoid_subset(a, b)
} by {
    if a.as_set.subset(b.as_set) {
        add_submonoid_subset_as_set_eq(a, b)
        add_submonoid_subset(a, b)
    }
}

/// Additive submonoid inclusion is reflexive.
theorem add_submonoid_subset_refl[M: AddMonoid](a: AddSubmonoid[M]) {
    add_submonoid_subset(a, a)
} by {
    forall(x: M) {
        a.contains(x) implies a.contains(x)
    }
}

/// Additive submonoid inclusion is transitive.
theorem add_submonoid_subset_trans[M: AddMonoid](
    a: AddSubmonoid[M],
    b: AddSubmonoid[M],
    c: AddSubmonoid[M]
) {
    add_submonoid_subset(a, b) and add_submonoid_subset(b, c) implies add_submonoid_subset(a, c)
} by {
    if add_submonoid_subset(a, b) and add_submonoid_subset(b, c) {
        forall(x: M) {
            if a.contains(x) {
                b.contains(x)
                c.contains(x)
            }
        }
    }
}

/// The intersection of two additive submonoids is contained in the left additive submonoid.
theorem add_submonoid_intersection_subset_left_relation[M: AddMonoid](
    a: AddSubmonoid[M],
    b: AddSubmonoid[M]
) {
    add_submonoid_subset(a.intersection(b), a)
} by {
    forall(x: M) {
        if a.intersection(b).contains(x) {
            add_submonoid_intersection_contains_eq(a, b, x)
            a.contains(x)
        }
    }
}

/// The intersection of two additive submonoids is contained in the right additive submonoid.
theorem add_submonoid_intersection_subset_right_relation[M: AddMonoid](
    a: AddSubmonoid[M],
    b: AddSubmonoid[M]
) {
    add_submonoid_subset(a.intersection(b), b)
} by {
    forall(x: M) {
        if a.intersection(b).contains(x) {
            add_submonoid_intersection_contains_eq(a, b, x)
            b.contains(x)
        }
    }
}

/// Every common additive submonoid of two additive submonoids is contained in their intersection.
theorem add_submonoid_subset_intersection_of_subset_left_right[M: AddMonoid](
    c: AddSubmonoid[M],
    a: AddSubmonoid[M],
    b: AddSubmonoid[M]
) {
    add_submonoid_subset(c, a) and add_submonoid_subset(c, b) implies add_submonoid_subset(c, a.intersection(b))
} by {
    if add_submonoid_subset(c, a) and add_submonoid_subset(c, b) {
        forall(x: M) {
            if c.contains(x) {
                a.contains(x)
                b.contains(x)
                add_submonoid_intersection_contains_eq(a, b, x)
                a.intersection(b).contains(x)
            }
        }
    }
}

/// Containment in an intersection is equivalent to containment in both additive submonoids.
theorem add_submonoid_subset_intersection_iff[M: AddMonoid](
    c: AddSubmonoid[M],
    a: AddSubmonoid[M],
    b: AddSubmonoid[M]
) {
    add_submonoid_subset(c, a.intersection(b)) = (add_submonoid_subset(c, a) and add_submonoid_subset(c, b))
} by {
    if add_submonoid_subset(c, a.intersection(b)) {
        add_submonoid_intersection_subset_left_relation(a, b)
        add_submonoid_subset_trans(c, a.intersection(b), a)
        add_submonoid_intersection_subset_right_relation(a, b)
        add_submonoid_subset_trans(c, a.intersection(b), b)
        add_submonoid_subset(c, b)
        add_submonoid_subset(c, a) and add_submonoid_subset(c, b)
    }
    if add_submonoid_subset(c, a) and add_submonoid_subset(c, b) {
        add_submonoid_subset_intersection_of_subset_left_right(c, a, b)
        add_submonoid_subset(c, a.intersection(b))
    }
}

/// True if an element is the additive identity.
define is_add_monoid_identity[M: AddMonoid](m: M) -> Bool {
    m = M.0
}

/// The additive identity predicate is an additive submonoid predicate.
theorem zero_add_submonoid_constraint[M: AddMonoid] {
    add_submonoid_constraint(is_add_monoid_identity[M])
} by {
    forall(a: M, b: M) {
        if is_add_monoid_identity(a) and is_add_monoid_identity(b) {
            b = M.0
            is_add_monoid_identity(a + b)
        }
    }
    add_submonoid_closure_constraint(is_add_monoid_identity[M])
}

/// The additive submonoid containing only the additive identity.
let zero_add_submonoid[M: AddMonoid]: AddSubmonoid[M] satisfy {
    AddSubmonoid.new(is_add_monoid_identity[M]) = Option.some(zero_add_submonoid)
}

/// Only the additive identity belongs to the zero additive submonoid.
theorem zero_add_submonoid_only_has_zero[M: AddMonoid](m: M) {
    zero_add_submonoid[M].contains(m) implies m = M.0
} by {
    if zero_add_submonoid[M].contains(m) {
        zero_add_submonoid[M].contains(m) = is_add_monoid_identity(m)
        m = M.0
    }
}

/// Membership in the zero additive submonoid is equality to the additive identity.
theorem zero_add_submonoid_contains_eq[M: AddMonoid](m: M) {
    zero_add_submonoid[M].contains(m) = (m = M.0)
} by {
    if zero_add_submonoid[M].contains(m) {
        zero_add_submonoid_only_has_zero(m)
    }
    if m = M.0 {
        add_submonoid_contains_zero(zero_add_submonoid[M])
        zero_add_submonoid[M].contains(m)
    }
}

/// The zero additive submonoid is contained in every additive submonoid.
theorem zero_add_submonoid_subset[M: AddMonoid](s: AddSubmonoid[M]) {
    add_submonoid_subset(zero_add_submonoid[M], s)
} by {
    forall(x: M) {
        if zero_add_submonoid[M].contains(x) {
            zero_add_submonoid_only_has_zero(x)
            add_submonoid_contains_zero(s)
            s.contains(x)
        }
    }
}

/// The full subset of any additive monoid is an additive submonoid.
define full_add_monoid_contains[M: AddMonoid](a: M) -> Bool {
    true
}

/// The full membership predicate is an additive submonoid predicate.
theorem full_add_submonoid_constraint[M: AddMonoid] {
    add_submonoid_constraint(full_add_monoid_contains[M])
} by {
    forall(a: M, b: M) {
        if full_add_monoid_contains[M](a) and full_add_monoid_contains[M](b) {
            full_add_monoid_contains[M](a + b)
        }
    }
    add_submonoid_closure_constraint(full_add_monoid_contains[M])
}

/// The full additive submonoid, containing every element.
let full_add_submonoid[M: AddMonoid]: AddSubmonoid[M] satisfy {
    AddSubmonoid.new(full_add_monoid_contains[M]) = Option.some(full_add_submonoid)
}

/// Every element belongs to the full additive submonoid.
theorem full_add_submonoid_contains_everything[M: AddMonoid](m: M) {
    full_add_submonoid[M].contains(m)
}

/// Membership in the full additive submonoid is always true.
theorem full_add_submonoid_contains_eq[M: AddMonoid](m: M) {
    full_add_submonoid[M].contains(m) = true
} by {
    full_add_submonoid_contains_everything(m)
}

/// Every additive submonoid is contained in the full additive submonoid.
theorem add_submonoid_subset_full[M: AddMonoid](s: AddSubmonoid[M]) {
    add_submonoid_subset(s, full_add_submonoid[M])
} by {
    forall(x: M) {
        if s.contains(x) {
            full_add_submonoid_contains_everything(x)
        }
    }
}

/// Mutual inclusion of additive submonoids forces equality.
theorem add_submonoid_subset_antisymm[M: AddMonoid](a: AddSubmonoid[M], b: AddSubmonoid[M]) {
    add_submonoid_subset(a, b) and add_submonoid_subset(b, a) implies a = b
} by {
    if add_submonoid_subset(a, b) and add_submonoid_subset(b, a) {
        add_submonoid_subset(a, b) = forall(y: M) {
            a.contains(y) implies b.contains(y)
        }
        add_submonoid_subset(b, a) = forall(y: M) {
            b.contains(y) implies a.contains(y)
        }
        forall(x: M) {
            if a.contains(x) {
                b.contains(x)
            }
            if b.contains(x) {
                a.contains(x)
            }
            a.contains(x) = b.contains(x)
        }
        predicate_extensionality(a.contains, b.contains)
        a = b
    }
}

/// Equality of additive submonoids is equivalent to mutual inclusion.
theorem add_submonoid_eq_iff_subset_both[M: AddMonoid](a: AddSubmonoid[M], b: AddSubmonoid[M]) {
    a = b = (add_submonoid_subset(a, b) and add_submonoid_subset(b, a))
} by {
    if a = b {
        add_submonoid_subset_refl(a)
        add_submonoid_subset(b, a)
        add_submonoid_subset(a, b) and add_submonoid_subset(b, a)
    }
    if add_submonoid_subset(a, b) and add_submonoid_subset(b, a) {
        add_submonoid_subset_antisymm(a, b)
        a = b
    }
}

/// Intersecting with a larger additive submonoid gives the smaller additive submonoid.
theorem add_submonoid_intersection_eq_left_of_subset[M: AddMonoid](
    a: AddSubmonoid[M],
    b: AddSubmonoid[M]
) {
    add_submonoid_subset(a, b) implies a.intersection(b) = a
} by {
    if add_submonoid_subset(a, b) {
        add_submonoid_intersection_subset_left_relation(a, b)
        add_submonoid_subset_intersection_of_subset_left_right(a, a, b)
        add_submonoid_subset(a, a.intersection(b))
        add_submonoid_subset_antisymm(a.intersection(b), a)
        a.intersection(b) = a
    }
}

/// Intersecting with a larger additive submonoid on the left gives the smaller additive submonoid.
theorem add_submonoid_intersection_eq_right_of_subset[M: AddMonoid](
    a: AddSubmonoid[M],
    b: AddSubmonoid[M]
) {
    add_submonoid_subset(b, a) implies a.intersection(b) = b
} by {
    if add_submonoid_subset(b, a) {
        add_submonoid_intersection_subset_right_relation(a, b)
        add_submonoid_subset_intersection_of_subset_left_right(b, a, b)
        add_submonoid_subset_antisymm(a.intersection(b), b)
        a.intersection(b) = b
    }
}

/// Intersecting the zero additive submonoid on the left gives the zero additive submonoid.
theorem zero_add_submonoid_intersection_left[M: AddMonoid](s: AddSubmonoid[M]) {
    zero_add_submonoid[M].intersection(s) = zero_add_submonoid[M]
} by {
    zero_add_submonoid_subset(s)
    add_submonoid_intersection_eq_left_of_subset(zero_add_submonoid[M], s)
}

/// Intersecting the zero additive submonoid on the right gives the zero additive submonoid.
theorem zero_add_submonoid_intersection_right[M: AddMonoid](s: AddSubmonoid[M]) {
    s.intersection(zero_add_submonoid[M]) = zero_add_submonoid[M]
} by {
    zero_add_submonoid_subset(s)
    add_submonoid_intersection_eq_right_of_subset(s, zero_add_submonoid[M])
}

/// Intersecting the full additive submonoid on the left gives the other additive submonoid.
theorem full_add_submonoid_intersection_left[M: AddMonoid](s: AddSubmonoid[M]) {
    full_add_submonoid[M].intersection(s) = s
} by {
    add_submonoid_subset_full(s)
    add_submonoid_intersection_eq_right_of_subset(full_add_submonoid[M], s)
}

/// Intersecting the full additive submonoid on the right gives the other additive submonoid.
theorem full_add_submonoid_intersection_right[M: AddMonoid](s: AddSubmonoid[M]) {
    s.intersection(full_add_submonoid[M]) = s
} by {
    add_submonoid_subset_full(s)
    add_submonoid_intersection_eq_left_of_subset(s, full_add_submonoid[M])
}

/// True if a set is contained in an additive submonoid.
define set_subset_add_submonoid[M: AddMonoid](a: Set[M], s: AddSubmonoid[M]) -> Bool {
    forall(x: M) {
        a.contains(x) implies s.contains(x)
    }
}

/// Set containment in an additive submonoid is containment in its underlying set.
theorem set_subset_add_submonoid_as_set_eq[M: AddMonoid](a: Set[M], s: AddSubmonoid[M]) {
    set_subset_add_submonoid(a, s) = a.subset(s.as_set)
} by {
    if set_subset_add_submonoid(a, s) {
        forall(x: M) {
            if a.contains(x) {
                s.contains(x)
                add_submonoid_as_set_contains_eq(s, x)
                s.as_set.contains(x)
            }
        }
        a.subset(s.as_set)
    }
    if a.subset(s.as_set) {
        forall(x: M) {
            if a.contains(x) {
                a.subset(s.as_set) = forall(y: M) {
                    a.contains(y) implies s.as_set.contains(y)
                }
                add_submonoid_as_set_contains_eq(s, x)
                s.contains(x)
            }
        }
        set_subset_add_submonoid(a, s)
    }
}

/// Set containment in an additive submonoid gives containment in its underlying set.
theorem set_as_set_subset_of_subset_add_submonoid[M: AddMonoid](a: Set[M], s: AddSubmonoid[M]) {
    set_subset_add_submonoid(a, s) implies a.subset(s.as_set)
} by {
    if set_subset_add_submonoid(a, s) {
        set_subset_add_submonoid_as_set_eq(a, s)
        a.subset(s.as_set)
    }
}

/// Containment in the underlying set gives set containment in the additive submonoid.
theorem set_subset_add_submonoid_of_as_set_subset[M: AddMonoid](a: Set[M], s: AddSubmonoid[M]) {
    a.subset(s.as_set) implies set_subset_add_submonoid(a, s)
} by {
    if a.subset(s.as_set) {
        set_subset_add_submonoid_as_set_eq(a, s)
        set_subset_add_submonoid(a, s)
    }
}

/// Set containment in an additive submonoid is reflexive for the underlying set.
theorem add_submonoid_as_set_subset_self[M: AddMonoid](s: AddSubmonoid[M]) {
    set_subset_add_submonoid(s.as_set, s)
} by {
    forall(x: M) {
        if s.as_set.contains(x) {
            s.contains(x)
        }
    }
}

/// Set containment is preserved by enlarging the target additive submonoid.
theorem set_subset_add_submonoid_trans[M: AddMonoid](
    a: Set[M],
    s: AddSubmonoid[M],
    t: AddSubmonoid[M]
) {
    set_subset_add_submonoid(a, s) and add_submonoid_subset(s, t) implies set_subset_add_submonoid(a, t)
} by {
    if set_subset_add_submonoid(a, s) and add_submonoid_subset(s, t) {
        forall(x: M) {
            if a.contains(x) {
                s.contains(x)
                t.contains(x)
            }
        }
    }
}

/// Set containment in an additive submonoid is preserved by enlarging the set.
theorem set_subset_add_submonoid_of_set_subset[M: AddMonoid](
    a: Set[M],
    b: Set[M],
    s: AddSubmonoid[M]
) {
    a.subset(b) and set_subset_add_submonoid(b, s) implies set_subset_add_submonoid(a, s)
} by {
    if a.subset(b) and set_subset_add_submonoid(b, s) {
        forall(x: M) {
            if a.contains(x) {
                a.subset(b) = forall(y: M) {
                    a.contains(y) implies b.contains(y)
                }
                b.contains(x)
                s.contains(x)
            }
        }
    }
}

/// True if an element belongs to every additive submonoid that contains a given set.
define add_submonoid_closure_contains[M: AddMonoid](a: Set[M], x: M) -> Bool {
    forall(s: AddSubmonoid[M]) {
        set_subset_add_submonoid(a, s) implies s.contains(x)
    }
}

/// Membership in the closure gives membership in each additive submonoid that contains the set.
theorem add_submonoid_closure_contains_of_set_subset_raw[M: AddMonoid](
    a: Set[M],
    s: AddSubmonoid[M],
    x: M
) {
    add_submonoid_closure_contains(a, x) and set_subset_add_submonoid(a, s) implies s.contains(x)
} by {
    if add_submonoid_closure_contains(a, x) and set_subset_add_submonoid(a, s) {
        add_submonoid_closure_contains(a, x) = forall(t: AddSubmonoid[M]) {
            set_subset_add_submonoid(a, t) implies t.contains(x)
        }
        s.contains(x)
    }
}

/// The additive submonoid closure of a set contains zero.
theorem add_submonoid_closure_zero_constraint[M: AddMonoid](a: Set[M]) {
    add_submonoid_zero_constraint(add_submonoid_closure_contains(a))
} by {
    forall(s: AddSubmonoid[M]) {
        if set_subset_add_submonoid(a, s) {
            add_submonoid_contains_zero(s)
            s.contains(M.0)
        }
    }
    add_submonoid_closure_contains(a, M.0)
}

/// The additive submonoid closure of a set is closed under addition.
theorem add_submonoid_closure_closure_constraint[M: AddMonoid](a: Set[M]) {
    add_submonoid_closure_constraint(add_submonoid_closure_contains(a))
} by {
    forall(x: M, y: M) {
        if add_submonoid_closure_contains(a, x) and add_submonoid_closure_contains(a, y) {
            forall(s: AddSubmonoid[M]) {
                if set_subset_add_submonoid(a, s) {
                    add_submonoid_closure_contains_of_set_subset_raw(a, s, x)
                    add_submonoid_closure_contains_of_set_subset_raw(a, s, y)
                    add_submonoid_contains_add(s, x, y)
                    s.contains(x + y)
                }
            }
            add_submonoid_closure_contains(a, x + y)
        }
    }
}

/// The additive submonoid generated by a set satisfies the additive submonoid laws.
theorem add_submonoid_generated_constraint[M: AddMonoid](a: Set[M]) {
    add_submonoid_constraint(add_submonoid_closure_contains(a))
} by {
    add_submonoid_closure_zero_constraint(a)
    add_submonoid_closure_closure_constraint(a)
}

/// The smallest additive submonoid containing a set.
let add_submonoid_closure[M: AddMonoid](a: Set[M]) -> result: AddSubmonoid[M] satisfy {
    AddSubmonoid.new(add_submonoid_closure_contains(a)) = Option.some(result)
} by {
    add_submonoid_generated_constraint(a)
}

/// Membership in the closure means membership in every additive submonoid containing the set.
theorem add_submonoid_closure_contains_eq[M: AddMonoid](a: Set[M], x: M) {
    add_submonoid_closure(a).contains(x) = add_submonoid_closure_contains(a, x)
} by {
}

/// Every generator belongs to the additive submonoid closure.
theorem add_submonoid_subset_closure[M: AddMonoid](a: Set[M]) {
    set_subset_add_submonoid(a, add_submonoid_closure(a))
} by {
    forall(x: M) {
        if a.contains(x) {
            forall(s: AddSubmonoid[M]) {
                if set_subset_add_submonoid(a, s) {
                    s.contains(x)
                }
            }
            add_submonoid_closure_contains(a, x)
            add_submonoid_closure_contains_eq(a, x)
            add_submonoid_closure(a).contains(x)
        }
    }
}

/// A generator is a member of the additive submonoid closure.
theorem add_submonoid_closure_contains_of_set_contains[M: AddMonoid](a: Set[M], x: M) {
    a.contains(x) implies add_submonoid_closure(a).contains(x)
} by {
    if a.contains(x) {
        add_submonoid_subset_closure(a)
        add_submonoid_closure(a).contains(x)
    }
}

/// The additive submonoid closure is contained in any additive submonoid containing the set.
theorem add_submonoid_closure_subset_of_set_subset[M: AddMonoid](a: Set[M], s: AddSubmonoid[M]) {
    set_subset_add_submonoid(a, s) implies add_submonoid_subset(add_submonoid_closure(a), s)
} by {
    if set_subset_add_submonoid(a, s) {
        forall(x: M) {
            if add_submonoid_closure(a).contains(x) {
                add_submonoid_closure_contains_eq(a, x)
                add_submonoid_closure_contains(a, x)
                add_submonoid_closure_contains_of_set_subset_raw(a, s, x)
            }
        }
    }
}

/// The additive submonoid closure is the least additive submonoid containing the set.
theorem add_submonoid_closure_le_iff_set_subset[M: AddMonoid](a: Set[M], s: AddSubmonoid[M]) {
    add_submonoid_subset(add_submonoid_closure(a), s) = set_subset_add_submonoid(a, s)
} by {
    if add_submonoid_subset(add_submonoid_closure(a), s) {
        add_submonoid_subset_closure(a)
        set_subset_add_submonoid_trans(a, add_submonoid_closure(a), s)
        set_subset_add_submonoid(a, s)
    }
    if set_subset_add_submonoid(a, s) {
        add_submonoid_closure_subset_of_set_subset(a, s)
        add_submonoid_subset(add_submonoid_closure(a), s)
    }
}

/// The additive submonoid closure and underlying set maps form a set-containment adjunction.
theorem add_submonoid_closure_as_set_galois_connection[M: AddMonoid](a: Set[M], s: AddSubmonoid[M]) {
    add_submonoid_closure(a).as_set.subset(s.as_set) = a.subset(s.as_set)
} by {
    add_submonoid_subset_as_set_eq(add_submonoid_closure(a), s)
    add_submonoid_closure_le_iff_set_subset(a, s)
    set_subset_add_submonoid_as_set_eq(a, s)
}

/// The closure of the underlying set of an additive submonoid is the additive submonoid itself.
theorem add_submonoid_closure_as_set[M: AddMonoid](s: AddSubmonoid[M]) {
    add_submonoid_closure(s.as_set) = s
} by {
    add_submonoid_as_set_subset_self(s)
    add_submonoid_closure_subset_of_set_subset(s.as_set, s)
    add_submonoid_subset_closure(s.as_set)
    forall(x: M) {
        if s.contains(x) {
            add_submonoid_closure(s.as_set).contains(x)
        }
    }
    add_submonoid_subset_antisymm(add_submonoid_closure(s.as_set), s)
}

/// Additive submonoid closure is monotone with respect to set inclusion.
theorem add_submonoid_closure_mono[M: AddMonoid](a: Set[M], b: Set[M]) {
    a.subset(b) implies add_submonoid_subset(add_submonoid_closure(a), add_submonoid_closure(b))
} by {
    if a.subset(b) {
        add_submonoid_subset_closure(b)
        set_subset_add_submonoid_of_set_subset(a, b, add_submonoid_closure(b))
        add_submonoid_closure_subset_of_set_subset(a, add_submonoid_closure(b))
        add_submonoid_subset(add_submonoid_closure(a), add_submonoid_closure(b))
    }
}

/// Equal sets have equal additive submonoid closures.
theorem add_submonoid_closure_eq_of_set_eq[M: AddMonoid](a: Set[M], b: Set[M]) {
    a = b implies add_submonoid_closure(a) = add_submonoid_closure(b)
} by {
    if a = b {
        add_submonoid_closure(a) = add_submonoid_closure(b)
    }
}

/// Applying additive submonoid closure twice gives the same additive submonoid.
theorem add_submonoid_closure_idempotent[M: AddMonoid](a: Set[M]) {
    add_submonoid_closure(add_submonoid_closure(a).as_set) = add_submonoid_closure(a)
} by {
    add_submonoid_closure_as_set(add_submonoid_closure(a))
}

/// The set closure induced by additive submonoid generation.
define add_submonoid_set_closure[M: AddMonoid](a: Set[M]) -> Set[M] {
    add_submonoid_closure(a).as_set
}

/// The additive submonoid set closure is the underlying set of the generated additive submonoid.
theorem add_submonoid_set_closure_at[M: AddMonoid](a: Set[M]) {
    add_submonoid_set_closure(a) = add_submonoid_closure(a).as_set
}

/// The additive submonoid set closure contains the original set.
theorem add_submonoid_set_closure_extensive[M: AddMonoid](a: Set[M]) {
    a.subset(add_submonoid_set_closure(a))
} by {
    add_submonoid_set_closure_at(a)
    add_submonoid_subset_closure(a)
    set_subset_add_submonoid_as_set_eq(a, add_submonoid_closure(a))
}

/// The additive submonoid set closure preserves inclusion.
theorem add_submonoid_set_closure_mono[M: AddMonoid](a: Set[M], b: Set[M]) {
    a.subset(b) implies add_submonoid_set_closure(a).subset(add_submonoid_set_closure(b))
} by {
    if a.subset(b) {
        add_submonoid_closure_mono(a, b)
        add_submonoid_subset_as_set_eq(add_submonoid_closure(a), add_submonoid_closure(b))
        add_submonoid_set_closure_at(a)
        add_submonoid_set_closure_at(b)
        add_submonoid_set_closure(a).subset(add_submonoid_set_closure(b))
    }
}

/// The additive submonoid set closure is unchanged after two applications.
theorem add_submonoid_set_closure_idempotent[M: AddMonoid](a: Set[M]) {
    add_submonoid_set_closure(add_submonoid_set_closure(a)) = add_submonoid_set_closure(a)
} by {
    add_submonoid_set_closure_at(a)
    add_submonoid_set_closure_at(add_submonoid_set_closure(a))
    add_submonoid_closure_idempotent(a)
}

/// Additive submonoid set closure is monotone as a set map.
theorem add_submonoid_set_closure_is_monotone[M: AddMonoid] {
    is_subset_monotone_map(add_submonoid_set_closure[M])
} by {
    forall(a: Set[M], b: Set[M]) {
        if a.subset(b) {
            add_submonoid_set_closure_mono(a, b)
            add_submonoid_set_closure(a).subset(add_submonoid_set_closure(b))
        }
    }
}

/// Additive submonoid set closure is extensive as a set map.
theorem add_submonoid_set_closure_is_extensive[M: AddMonoid] {
    is_set_extensive_map(add_submonoid_set_closure[M])
} by {
    forall(a: Set[M]) {
        add_submonoid_set_closure_extensive(a)
        a.subset(add_submonoid_set_closure(a))
    }
}

/// Additive submonoid set closure is idempotent as a set map.
theorem add_submonoid_set_closure_is_idempotent[M: AddMonoid] {
    is_set_idempotent_map(add_submonoid_set_closure[M])
} by {
    forall(a: Set[M]) {
        add_submonoid_set_closure_idempotent(a)
        add_submonoid_set_closure(add_submonoid_set_closure(a)) = add_submonoid_set_closure(a)
    }
}

/// Additive submonoid generation induces a closure operator on sets.
theorem add_submonoid_set_closure_operator[M: AddMonoid] {
    is_set_closure_operator(add_submonoid_set_closure[M])
} by {
    add_submonoid_set_closure_is_monotone[M]
    add_submonoid_set_closure_is_extensive[M]
    add_submonoid_set_closure_is_idempotent[M]
}

/// The closure of the empty set is the zero additive submonoid.
theorem add_submonoid_closure_empty[M: AddMonoid] {
    add_submonoid_closure(Set[M].empty_set) = zero_add_submonoid[M]
} by {
    set_subset_add_submonoid(Set[M].empty_set, zero_add_submonoid[M])
    add_submonoid_closure_subset_of_set_subset(Set[M].empty_set, zero_add_submonoid[M])
    zero_add_submonoid_subset(add_submonoid_closure(Set[M].empty_set))
    add_submonoid_subset_antisymm(add_submonoid_closure(Set[M].empty_set), zero_add_submonoid[M])
}

/// The closure of the universal set is the full additive submonoid.
theorem add_submonoid_closure_universal[M: AddMonoid] {
    add_submonoid_closure(Set[M].universal_set) = full_add_submonoid[M]
} by {
    add_submonoid_subset_full(add_submonoid_closure(Set[M].universal_set))
    forall(x: M) {
        if full_add_submonoid[M].contains(x) {
            add_submonoid_closure_contains_of_set_contains(Set[M].universal_set, x)
            add_submonoid_closure(Set[M].universal_set).contains(x)
        }
    }
    add_submonoid_subset(full_add_submonoid[M], add_submonoid_closure(Set[M].universal_set))
    add_submonoid_subset_antisymm(add_submonoid_closure(Set[M].universal_set), full_add_submonoid[M])
}

/// The least additive submonoid containing two additive submonoids.
define add_submonoid_sup[M: AddMonoid](a: AddSubmonoid[M], b: AddSubmonoid[M]) -> AddSubmonoid[M] {
    add_submonoid_closure(a.as_set.union(b.as_set))
}

/// The join of two additive submonoids is the closure of the union of their underlying sets.
theorem add_submonoid_sup_eq_closure_union[M: AddMonoid](a: AddSubmonoid[M], b: AddSubmonoid[M]) {
    add_submonoid_sup(a, b) = add_submonoid_closure(a.as_set.union(b.as_set))
}

/// The left additive submonoid is contained in the join.
theorem add_submonoid_subset_sup_left[M: AddMonoid](a: AddSubmonoid[M], b: AddSubmonoid[M]) {
    add_submonoid_subset(a, add_submonoid_sup(a, b))
} by {
    forall(x: M) {
        if a.contains(x) {
            add_submonoid_as_set_contains_eq(a, x)
            union_contains_left(a.as_set, b.as_set, x)
            add_submonoid_closure_contains_of_set_contains(a.as_set.union(b.as_set), x)
            add_submonoid_sup(a, b).contains(x)
        }
    }
}

/// The right additive submonoid is contained in the join.
theorem add_submonoid_subset_sup_right[M: AddMonoid](a: AddSubmonoid[M], b: AddSubmonoid[M]) {
    add_submonoid_subset(b, add_submonoid_sup(a, b))
} by {
    forall(x: M) {
        if b.contains(x) {
            add_submonoid_as_set_contains_eq(b, x)
            union_contains_right(a.as_set, b.as_set, x)
            add_submonoid_closure_contains_of_set_contains(a.as_set.union(b.as_set), x)
            add_submonoid_sup(a, b).contains(x)
        }
    }
}

/// The join is contained in every common upper bound.
theorem add_submonoid_sup_subset_of_subset_left_right[M: AddMonoid](
    a: AddSubmonoid[M],
    b: AddSubmonoid[M],
    c: AddSubmonoid[M]
) {
    add_submonoid_subset(a, c) and add_submonoid_subset(b, c) implies
        add_submonoid_subset(add_submonoid_sup(a, b), c)
} by {
    if add_submonoid_subset(a, c) and add_submonoid_subset(b, c) {
        forall(x: M) {
            if a.as_set.union(b.as_set).contains(x) {
                union_contains_eq(a.as_set, b.as_set, x)
                if a.as_set.contains(x) {
                    add_submonoid_as_set_contains_eq(a, x)
                    a.contains(x)
                    c.contains(x)
                } else {
                    b.as_set.contains(x)
                    add_submonoid_as_set_contains_eq(b, x)
                    b.contains(x)
                    c.contains(x)
                }
            }
        }
        set_subset_add_submonoid(a.as_set.union(b.as_set), c)
        add_submonoid_closure_subset_of_set_subset(a.as_set.union(b.as_set), c)
        add_submonoid_subset(add_submonoid_sup(a, b), c)
    }
}

/// Containment of a join is equivalent to containment of both additive submonoids.
theorem add_submonoid_sup_subset_iff[M: AddMonoid](
    a: AddSubmonoid[M],
    b: AddSubmonoid[M],
    c: AddSubmonoid[M]
) {
    add_submonoid_subset(add_submonoid_sup(a, b), c) =
        (add_submonoid_subset(a, c) and add_submonoid_subset(b, c))
} by {
    if add_submonoid_subset(add_submonoid_sup(a, b), c) {
        add_submonoid_subset_sup_left(a, b)
        add_submonoid_subset_trans(a, add_submonoid_sup(a, b), c)
        add_submonoid_subset_sup_right(a, b)
        add_submonoid_subset_trans(b, add_submonoid_sup(a, b), c)
        add_submonoid_subset(b, c)
        add_submonoid_subset(a, c) and add_submonoid_subset(b, c)
    }
    if add_submonoid_subset(a, c) and add_submonoid_subset(b, c) {
        add_submonoid_sup_subset_of_subset_left_right(a, b, c)
        add_submonoid_subset(add_submonoid_sup(a, b), c)
    }
}

attributes AddSubmonoid[M: AddMonoid] {
    /// The smallest additive submonoid containing a set.
    let closure: Set[M] -> AddSubmonoid[M] = add_submonoid_closure

    /// The least additive submonoid containing this additive submonoid and another additive submonoid.
    let sup: (AddSubmonoid[M], AddSubmonoid[M]) -> AddSubmonoid[M] = add_submonoid_sup
}

/// True if an additive submonoid is generated by a finite set.
define add_submonoid_is_finitely_generated[M: AddMonoid](s: AddSubmonoid[M]) -> Bool {
    exists(a: Set[M]) {
        a.is_finite and add_submonoid_closure(a) = s
    }
}

/// A finitely generated additive submonoid has a finite generating set.
theorem add_submonoid_is_finitely_generated_witness[M: AddMonoid](s: AddSubmonoid[M]) {
    add_submonoid_is_finitely_generated(s) implies exists(a: Set[M]) {
        a.is_finite and add_submonoid_closure(a) = s
    }
}

/// An additive submonoid equal to the closure of a finite set is finitely generated.
theorem add_submonoid_is_finitely_generated_of_closure_eq[M: AddMonoid](
    a: Set[M],
    s: AddSubmonoid[M]
) {
    a.is_finite and add_submonoid_closure(a) = s implies add_submonoid_is_finitely_generated(s)
} by {
    if a.is_finite and add_submonoid_closure(a) = s {
        add_submonoid_is_finitely_generated(s)
    }
}

/// Equality preserves finite generation of additive submonoids.
theorem add_submonoid_is_finitely_generated_of_eq[M: AddMonoid](
    s: AddSubmonoid[M],
    t: AddSubmonoid[M]
) {
    add_submonoid_is_finitely_generated(s) and s = t implies add_submonoid_is_finitely_generated(t)
} by {
    if add_submonoid_is_finitely_generated(s) and s = t {
        add_submonoid_is_finitely_generated_witness(s)
        let a: Set[M] satisfy {
            a.is_finite and add_submonoid_closure(a) = s
        }
        add_submonoid_is_finitely_generated_of_closure_eq(a, t)
        add_submonoid_is_finitely_generated(t)
    }
}

/// The closure of a finite set is a finitely generated additive submonoid.
theorem add_submonoid_closure_is_finitely_generated[M: AddMonoid](a: Set[M]) {
    a.is_finite implies add_submonoid_is_finitely_generated(add_submonoid_closure(a))
} by {
    if a.is_finite {
        add_submonoid_is_finitely_generated_of_closure_eq(a, add_submonoid_closure(a))
    }
}

/// The inverse image membership predicate of an additive submonoid under an additive monoid homomorphism.
define add_submonoid_preimage_contains[A: AddMonoid, B: AddMonoid](
    f: AddMonoidHom[A, B],
    s: AddSubmonoid[B],
    a: A
) -> Bool {
    s.contains(f.hom(a))
}

/// The inverse image of an additive submonoid contains zero.
theorem add_submonoid_preimage_zero_constraint[A: AddMonoid, B: AddMonoid](
    f: AddMonoidHom[A, B],
    s: AddSubmonoid[B]
) {
    add_submonoid_zero_constraint(add_submonoid_preimage_contains(f, s))
} by {
    add_monoid_hom_zero(f)
    add_submonoid_contains_zero(s)
}

/// The inverse image of an additive submonoid is closed under addition.
theorem add_submonoid_preimage_closure_constraint[A: AddMonoid, B: AddMonoid](
    f: AddMonoidHom[A, B],
    s: AddSubmonoid[B]
) {
    add_submonoid_closure_constraint(add_submonoid_preimage_contains(f, s))
} by {
    forall(a: A, b: A) {
        if add_submonoid_preimage_contains(f, s, a) and add_submonoid_preimage_contains(f, s, b) {
            s.contains(f.hom(b))
            add_submonoid_contains_add(s, f.hom(a), f.hom(b))
            s.contains(f.hom(a) + f.hom(b))
            add_monoid_hom_add(f, a, b)
            add_submonoid_preimage_contains(f, s, a + b)
        }
    }
}

/// The inverse image of an additive submonoid is an additive submonoid.
theorem add_submonoid_preimage_constraint[A: AddMonoid, B: AddMonoid](
    f: AddMonoidHom[A, B],
    s: AddSubmonoid[B]
) {
    add_submonoid_constraint(add_submonoid_preimage_contains(f, s))
} by {
    add_submonoid_preimage_zero_constraint(f, s)
    add_submonoid_preimage_closure_constraint(f, s)
}

/// The inverse image of an additive submonoid under an additive monoid homomorphism.
let add_submonoid_preimage[A: AddMonoid, B: AddMonoid](f: AddMonoidHom[A, B], s: AddSubmonoid[B]) -> result: AddSubmonoid[A] satisfy {
    AddSubmonoid.new(add_submonoid_preimage_contains(f, s)) = Option.some(result)
} by {
    add_submonoid_preimage_constraint(f, s)
}

/// Membership in an inverse image means the mapped element belongs to the target additive submonoid.
theorem add_submonoid_preimage_contains_eq[A: AddMonoid, B: AddMonoid](
    f: AddMonoidHom[A, B],
    s: AddSubmonoid[B],
    a: A
) {
    add_submonoid_preimage(f, s).contains(a) = s.contains(f.hom(a))
} by {
    add_submonoid_preimage(f, s).contains(a) = add_submonoid_preimage_contains(f, s, a)
}

/// Membership in a preimage follows from membership of the mapped element.
theorem add_submonoid_preimage_contains_of_contains_map[A: AddMonoid, B: AddMonoid](
    f: AddMonoidHom[A, B],
    s: AddSubmonoid[B],
    a: A
) {
    s.contains(f.hom(a)) implies add_submonoid_preimage(f, s).contains(a)
} by {
    if s.contains(f.hom(a)) {
        add_submonoid_preimage_contains_eq(f, s, a)
        add_submonoid_preimage(f, s).contains(a)
    }
}

/// Membership in the target follows from membership in the preimage.
theorem add_submonoid_contains_map_of_preimage_contains[A: AddMonoid, B: AddMonoid](
    f: AddMonoidHom[A, B],
    s: AddSubmonoid[B],
    a: A
) {
    add_submonoid_preimage(f, s).contains(a) implies s.contains(f.hom(a))
} by {
    if add_submonoid_preimage(f, s).contains(a) {
        add_submonoid_preimage_contains_eq(f, s, a)
        s.contains(f.hom(a))
    }
}

/// Inverse images are monotone with respect to additive submonoid containment.
theorem add_submonoid_preimage_subset_of_subset[A: AddMonoid, B: AddMonoid](
    f: AddMonoidHom[A, B],
    s: AddSubmonoid[B],
    t: AddSubmonoid[B]
) {
    add_submonoid_subset(s, t) implies
        add_submonoid_subset(add_submonoid_preimage(f, s), add_submonoid_preimage(f, t))
} by {
    if add_submonoid_subset(s, t) {
        forall(a: A) {
            if add_submonoid_preimage(f, s).contains(a) {
                add_submonoid_preimage_contains_eq(f, s, a)
                add_submonoid_subset(s, t) = forall(x: B) {
                    s.contains(x) implies t.contains(x)
                }
                add_submonoid_preimage_contains_eq(f, t, a)
                add_submonoid_preimage(f, t).contains(a)
            }
        }
    }
}

/// The inverse image of an intersection is the intersection of the inverse images.
theorem add_submonoid_preimage_intersection[A: AddMonoid, B: AddMonoid](
    f: AddMonoidHom[A, B],
    s: AddSubmonoid[B],
    t: AddSubmonoid[B]
) {
    add_submonoid_preimage(f, s.intersection(t)) =
        add_submonoid_preimage(f, s).intersection(add_submonoid_preimage(f, t))
} by {
    forall(a: A) {
        add_submonoid_preimage_contains_eq(f, s.intersection(t), a)
        add_submonoid_preimage_contains_eq(f, s, a)
        add_submonoid_preimage_contains_eq(f, t, a)
        add_submonoid_intersection_contains_eq(s, t, f.hom(a))
        add_submonoid_intersection_contains_eq(add_submonoid_preimage(f, s), add_submonoid_preimage(f, t), a)
        add_submonoid_preimage(f, s.intersection(t)).contains(a) =
            add_submonoid_preimage(f, s).intersection(add_submonoid_preimage(f, t)).contains(a)
    }
    add_submonoid_ext(add_submonoid_preimage(f, s.intersection(t)),
        add_submonoid_preimage(f, s).intersection(add_submonoid_preimage(f, t)))
}

/// The inverse image of the full additive submonoid is the full additive submonoid.
theorem add_submonoid_preimage_full[A: AddMonoid, B: AddMonoid](f: AddMonoidHom[A, B]) {
    add_submonoid_preimage(f, full_add_submonoid[B]) = full_add_submonoid[A]
} by {
    add_submonoid_subset_full(add_submonoid_preimage(f, full_add_submonoid[B]))
    forall(a: A) {
        if full_add_submonoid[A].contains(a) {
            full_add_submonoid_contains_everything(f.hom(a))
            add_submonoid_preimage_contains_eq(f, full_add_submonoid[B], a)
            add_submonoid_preimage(f, full_add_submonoid[B]).contains(a)
        }
    }
    add_submonoid_subset(full_add_submonoid[A], add_submonoid_preimage(f, full_add_submonoid[B]))
    add_submonoid_subset_antisymm(add_submonoid_preimage(f, full_add_submonoid[B]), full_add_submonoid[A])
}

/// The kernel of an additive monoid homomorphism as an additive submonoid.
define add_monoid_hom_kernel[A: AddMonoid, B: AddMonoid](f: AddMonoidHom[A, B]) -> AddSubmonoid[A] {
    add_submonoid_preimage(f, zero_add_submonoid[B])
}

/// Membership in the kernel means the homomorphism maps the element to zero.
theorem add_monoid_hom_kernel_contains_eq[A: AddMonoid, B: AddMonoid](f: AddMonoidHom[A, B], a: A) {
    add_monoid_hom_kernel(f).contains(a) = (f.hom(a) = B.0)
} by {
    add_submonoid_preimage_contains_eq(f, zero_add_submonoid[B], a)
    if add_monoid_hom_kernel(f).contains(a) {
        zero_add_submonoid[B].contains(f.hom(a))
        zero_add_submonoid_only_has_zero(f.hom(a))
        f.hom(a) = B.0
    }
    if f.hom(a) = B.0 {
        add_submonoid_contains_zero(zero_add_submonoid[B])
        add_monoid_hom_kernel(f).contains(a)
    }
}

/// An element belongs to the kernel when its image is zero.
theorem add_monoid_hom_kernel_contains_of_maps_to_zero[A: AddMonoid, B: AddMonoid](
    f: AddMonoidHom[A, B],
    a: A
) {
    f.hom(a) = B.0 implies add_monoid_hom_kernel(f).contains(a)
} by {
    if f.hom(a) = B.0 {
        add_monoid_hom_kernel_contains_eq(f, a)
        add_monoid_hom_kernel(f).contains(a)
    }
}

/// A kernel element maps to zero.
theorem add_monoid_hom_maps_to_zero_of_kernel_contains[A: AddMonoid, B: AddMonoid](
    f: AddMonoidHom[A, B],
    a: A
) {
    add_monoid_hom_kernel(f).contains(a) implies f.hom(a) = B.0
} by {
    if add_monoid_hom_kernel(f).contains(a) {
        add_monoid_hom_kernel_contains_eq(f, a)
        f.hom(a) = B.0
    }
}


/// Additive submonoid join is commutative.
theorem add_submonoid_sup_comm[M: AddMonoid](a: AddSubmonoid[M], b: AddSubmonoid[M]) {
    add_submonoid_sup(a, b) = add_submonoid_sup(b, a)
} by {
    add_submonoid_subset_sup_right(b, a)
    add_submonoid_subset_sup_left(b, a)
    add_submonoid_sup_subset_of_subset_left_right(a, b, add_submonoid_sup(b, a))
    add_submonoid_subset_sup_right(a, b)
    add_submonoid_subset_sup_left(a, b)
    add_submonoid_sup_subset_of_subset_left_right(b, a, add_submonoid_sup(a, b))
    add_submonoid_subset_antisymm(add_submonoid_sup(a, b), add_submonoid_sup(b, a))
}

/// Joining an additive submonoid with itself gives the same submonoid.
theorem add_submonoid_sup_idempotent[M: AddMonoid](a: AddSubmonoid[M]) {
    add_submonoid_sup(a, a) = a
} by {
    add_submonoid_subset_refl(a)
    add_submonoid_sup_subset_of_subset_left_right(a, a, a)
    add_submonoid_subset_sup_left(a, a)
    add_submonoid_subset_antisymm(add_submonoid_sup(a, a), a)
}

/// Additive submonoid join is associative.
theorem add_submonoid_sup_assoc[M: AddMonoid](
    a: AddSubmonoid[M],
    b: AddSubmonoid[M],
    c: AddSubmonoid[M]
) {
    add_submonoid_sup(add_submonoid_sup(a, b), c) = add_submonoid_sup(a, add_submonoid_sup(b, c))
} by {
    add_submonoid_subset_sup_left(a, add_submonoid_sup(b, c))
    add_submonoid_subset_sup_right(a, add_submonoid_sup(b, c))
    add_submonoid_subset_sup_left(b, c)
    add_submonoid_subset_sup_right(b, c)
    add_submonoid_subset_trans(b, add_submonoid_sup(b, c), add_submonoid_sup(a, add_submonoid_sup(b, c)))
    add_submonoid_subset_trans(c, add_submonoid_sup(b, c), add_submonoid_sup(a, add_submonoid_sup(b, c)))
    add_submonoid_sup_subset_of_subset_left_right(a, b, add_submonoid_sup(a, add_submonoid_sup(b, c)))
    add_submonoid_sup_subset_of_subset_left_right(add_submonoid_sup(a, b), c, add_submonoid_sup(a, add_submonoid_sup(b, c)))
    add_submonoid_subset_sup_left(add_submonoid_sup(a, b), c)
    add_submonoid_subset_sup_right(add_submonoid_sup(a, b), c)
    add_submonoid_subset_sup_left(a, b)
    add_submonoid_subset_sup_right(a, b)
    add_submonoid_subset_trans(a, add_submonoid_sup(a, b), add_submonoid_sup(add_submonoid_sup(a, b), c))
    add_submonoid_subset_trans(b, add_submonoid_sup(a, b), add_submonoid_sup(add_submonoid_sup(a, b), c))
    add_submonoid_sup_subset_of_subset_left_right(b, c, add_submonoid_sup(add_submonoid_sup(a, b), c))
    add_submonoid_sup_subset_of_subset_left_right(a, add_submonoid_sup(b, c), add_submonoid_sup(add_submonoid_sup(a, b), c))
    add_submonoid_subset_antisymm(add_submonoid_sup(add_submonoid_sup(a, b), c), add_submonoid_sup(a, add_submonoid_sup(b, c)))
}
