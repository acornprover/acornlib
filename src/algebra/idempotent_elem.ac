/// Element-level idempotence in multiplicative monoids.

from algebra.monoid.monoid import Monoid
from algebra.comm_monoid import CommMonoid
from algebra.comm_semigroup import CommSemigroup

/// An element of a monoid is idempotent when multiplying it by itself fixes it.
define is_idempotent_elem[M: Monoid](a: M) -> Bool {
    a * a = a
}

/// The predicate unfolds to the usual multiplication equation.
theorem is_idempotent_elem_eq[M: Monoid](a: M) {
    is_idempotent_elem(a) = (a * a = a)
} by {
    is_idempotent_elem(a) = (a * a = a)
}

/// The identity element is idempotent.
theorem one_is_idempotent_elem[M: Monoid] {
    is_idempotent_elem(M.1)
} by {
    M.1 * M.1 = M.1
    is_idempotent_elem(M.1) = (M.1 * M.1 = M.1)
}

/// An idempotent element satisfies the defining multiplication equation.
theorem idempotent_elem_mul_self[M: Monoid](a: M) {
    is_idempotent_elem(a) implies a * a = a
} by {
    if is_idempotent_elem(a) {
        is_idempotent_elem(a) = (a * a = a)
    }
}

/// The multiplication equation implies element idempotence.
theorem is_idempotent_elem_of_mul_self[M: Monoid](a: M) {
    a * a = a implies is_idempotent_elem(a)
} by {
    if a * a = a {
        is_idempotent_elem(a) = (a * a = a)
    }
}

/// The product of two commuting idempotents is idempotent.
theorem idempotent_elem_mul_of_commute[M: Monoid](a: M, b: M) {
    is_idempotent_elem(a) and is_idempotent_elem(b) and a * b = b * a implies
    is_idempotent_elem(a * b)
} by {
    if is_idempotent_elem(a) and is_idempotent_elem(b) and a * b = b * a {
        idempotent_elem_mul_self(a)
        idempotent_elem_mul_self(b)
        a * a = a
        b * b = b
        (a * b) * (a * b) = a * (b * a) * b
        b * a = a * b
        a * (b * a) * b = a * (a * b) * b
        a * (a * b) * b = (a * a) * (b * b)
        (a * a) * (b * b) = a * b
        (a * b) * (a * b) = a * b
        is_idempotent_elem(a * b)
    }
}

/// In a commutative monoid, the product of two idempotents is idempotent.
theorem idempotent_elem_mul_comm_monoid[M: CommMonoid](a: M, b: M) {
    is_idempotent_elem(a) and is_idempotent_elem(b) implies is_idempotent_elem(a * b)
} by {
    if is_idempotent_elem(a) and is_idempotent_elem(b) {
        CommSemigroup.commutative[M](a, b)
        a * b = b * a
        idempotent_elem_mul_of_commute(a, b)
    }
}
