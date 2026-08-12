// Integral-domain deepening: cancellation and zero-product laws stated as
// iffs, squares and powers vanishing only at zero, and the fact that the
// only nilpotent element of an integral domain is zero.

from algebra.integral_domain import IntegralDomain, integral_domain_mul_left_cancel,
    integral_domain_mul_right_cancel, integral_domain_mul_eq_zero, integral_domain_mul_eq_zero_of_factor,
    integral_domain_pow_nonzero
from algebra.ring.ring import Ring, mul_zero_left
from algebra.ring.ring_axioms_deep import ring_pow_two
from algebra.ring.ring_characteristic_deep import is_nilpotent, nilpotent_apply
from nat import Nat, pow_zero, alt_induction
numerals Nat

/// In an integral domain, multiplication by a nonzero element preserves and reflects equality.
theorem integral_domain_mul_left_cancel_iff[D: IntegralDomain](a: D, b: D, c: D) {
    a != D.0 implies ((a * b = a * c) = (b = c))
} by {
    if a != D.0 {
        if a * b = a * c {
            integral_domain_mul_left_cancel(a, b, c)
            b = c
        }
        if b = c {
            a * b = a * c
        }
        (a * b = a * c) = (b = c)
    }
}

/// In an integral domain, multiplication by a nonzero element preserves and reflects equality on the right.
theorem integral_domain_mul_right_cancel_iff[D: IntegralDomain](a: D, b: D, c: D) {
    a != D.0 implies ((b * a = c * a) = (b = c))
} by {
    if a != D.0 {
        if b * a = c * a {
            integral_domain_mul_right_cancel(a, b, c)
            b = c
        }
        if b = c {
            b * a = c * a
        }
        (b * a = c * a) = (b = c)
    }
}

/// In an integral domain, a product is zero exactly when one of the factors is zero.
theorem integral_domain_mul_eq_zero_iff[D: IntegralDomain](a: D, b: D) {
    (a * b = D.0) = (a = D.0 or b = D.0)
} by {
    if a * b = D.0 {
        integral_domain_mul_eq_zero(a, b)
        a = D.0 or b = D.0
    }
    if a = D.0 or b = D.0 {
        integral_domain_mul_eq_zero_of_factor(a, b)
        a * b = D.0
    }
    (a * b = D.0) = (a = D.0 or b = D.0)
}

/// In an integral domain, a square is zero exactly when the element is zero.
theorem integral_domain_sq_eq_zero_iff[D: IntegralDomain](a: D) {
    (a * a = D.0) = (a = D.0)
} by {
    if a * a = D.0 {
        integral_domain_mul_eq_zero(a, a)
        if a = D.0 {
            a = D.0
        } else {
            a = D.0
        }
        a = D.0
    }
    if a = D.0 {
        a * a = D.0 * D.0
        mul_zero_left(D.0)
        D.0 * D.0 = D.0
        a * a = D.0
    }
    (a * a = D.0) = (a = D.0)
}

/// In an integral domain, a positive power is zero exactly when the element is zero.
theorem integral_domain_pow_eq_zero_iff[D: IntegralDomain](a: D, n: Nat) {
    n != Nat.0 implies ((a.pow(n) = D.0) = (a = D.0))
} by {
    if n != Nat.0 {
        if a.pow(n) = D.0 {
            if a != D.0 {
                integral_domain_pow_nonzero(a, n)
                a.pow(n) != D.0
                false
            }
            a = D.0
        }
        if a = D.0 {
            define p(k: Nat) -> Bool {
                D.0.pow(k.suc) = D.0
            }
            ring_pow_two(D.0)
            D.0.pow(2) = D.0 * D.0
            mul_zero_left(D.0)
            D.0 * D.0 = D.0
            D.0.pow(2) = D.0
            D.0.pow(Nat.0.suc) = D.0
            p(Nat.0)
            forall(k: Nat) {
                if p(k) {
                    D.0.pow(k.suc.suc) = D.0 * D.0.pow(k.suc)
                    D.0.pow(k.suc) = D.0
                    D.0 * D.0.pow(k.suc) = D.0 * D.0
                    D.0 * D.0 = D.0
                    D.0.pow(k.suc.suc) = D.0
                    p(k.suc)
                }
            }
            p(Nat.0) and forall(k: Nat) { p(k) implies p(k.suc) }
            alt_induction(p)
            forall(k: Nat) { p(k) }
            let m: Nat satisfy { n = m.suc }
            p(m)
            D.0.pow(m.suc) = D.0
            D.0.pow(n) = D.0
            a.pow(n) = D.0
        }
        (a.pow(n) = D.0) = (a = D.0)
    }
}

/// The only nilpotent element of an integral domain is zero.
theorem integral_domain_nilpotent_zero[D: IntegralDomain](a: D) {
    is_nilpotent[D](a) implies a = D.0
} by {
    if is_nilpotent[D](a) {
        nilpotent_apply(a)
        exists(n: Nat) { n != Nat.0 and a.pow(n) = D.0 }
        let n: Nat satisfy { n != Nat.0 and a.pow(n) = D.0 }
        integral_domain_pow_eq_zero_iff(a, n)
        n != Nat.0 implies ((a.pow(n) = D.0) = (a = D.0))
        (a.pow(n) = D.0) = (a = D.0)
        a.pow(n) = D.0
        a = D.0
    }
}
