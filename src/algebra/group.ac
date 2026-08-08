from algebra.monoid.monoid import Monoid, MonoidHom, is_monoid_hom, identity_fn_is_monoid_hom,
    compose_is_monoid_hom
from nat import Nat, pow_zero
from data.basic.functions import identity_fn, compose, compose_assoc, compose_identity_left, compose_identity_right,
    function_eq_transport_predicate, function_eq_transport_predicate_rev
from data.basic.relation_transport import preserves_binary_op

/// A group is a monoid that also has inverses.
typeclass G: Group extends Monoid {
    /// The inverse operation
    inverse: G -> G

    /// We only need right-inverse; we can prove left-inverse from it.
    inverse_right(a: G) {
        a * a.inverse = G.1
    }
}

// This direction is proven rather than assumed
theorem inverse_left[G: Group](a: G) {
    a.inverse * a = G.1
} by {
    (a.inverse * a) * (a.inverse * a.inverse.inverse) = a.inverse * a.inverse.inverse
}

theorem inverse_inverse[G: Group](a: G) {
    a.inverse.inverse = a
} by {
    a * a.inverse * a.inverse.inverse = a.inverse.inverse
}

theorem left_cancel[G: Group](a: G, b: G, c: G) {
    a * b = a * c implies b = c
} by {
    a.inverse * (a * b) = a.inverse * a * b
    a.inverse * (a * c) = a.inverse * a * c
    a.inverse * a = G.1
}

theorem right_cancel[G: Group](a: G, b: G, c: G) {
    b * a = c * a implies b = c
} by {
    b * a * a.inverse = b * (a * a.inverse)
    c * a * a.inverse = c * (a * a.inverse)
}

theorem inverse_mul[G: Group](a: G, b: G) {
    (a * b).inverse = b.inverse * a.inverse
} by {
    (a * b).inverse * (a * b) = G.1
    a * a.inverse = G.1
    (a * b).inverse * G.1 = (a * b).inverse
    (a * b).inverse * a * b = (a * b).inverse * (a * b)
    (a * b).inverse * a = b.inverse
}

theorem pow_inverse[G: Group](a: G, n: Nat) {
    a.pow(n).inverse = a.inverse.pow(n)
} by {
    define p(m: Nat) -> Bool {
        a.pow(m).inverse = a.inverse.pow(m)
    }

    a.pow(Nat.0) = G.1
    a.inverse.pow(Nat.0) = G.1
    G.1.inverse * G.1 = G.1
    G.1.inverse * G.1 = G.1.inverse
    G.1.inverse = G.1
    p(Nat.0)

    forall(k: Nat) {
        if p(k) {
            a.inverse.pow(k) = a.pow(k).inverse
            a.inverse.pow(k) * a.inverse.pow(Nat.1) = a.inverse.pow(k + Nat.1)
            a.pow(k).inverse * a.inverse = (a * a.pow(k)).inverse
            a.pow(k.suc).inverse = a.inverse.pow(k.suc)
            p(k.suc)
        }
    }
}

theorem pow_cancel[G: Group](a: G, m: Nat, n: Nat) {
    n > m and a.pow(n) = a.pow(m) implies a.pow(n - m) = G.1
} by {
    a.pow(n) = a.pow(n - m)*a.pow(m)
}

/// True if an element has finite order (some positive power equals the identity).
define has_finite_order[G: Group](g: G) -> Bool {
    exists(n: Nat) {
        n != Nat.0 and g.pow(n) = G.1
    }
}

/// True if a group has no elements of finite order except the identity.
let is_torsion_free[G: Group] = forall(g: G) {
    has_finite_order(g) implies g = G.1
}

/// True if a function preserves the group operation (is a homomorphism).
define is_group_hom[G: Group, H: Group](f: G -> H) -> Bool {
    forall(a: G, b: G) {
        f(a * b) = f(a) * f(b)
    }
}

/// The trivial group homomorphism maps every element of G to the identity element of H.
let trivial_group_hom[G: Group, H: Group]: G -> H = function(a: G) {
    H.1
}

/// The trivial group homomorphism preserves the group operation.
theorem trivial_group_hom_is_hom[G: Group, H: Group] {
    is_group_hom(trivial_group_hom[G, H])
}

/// A group homomorphism that preserves the group structure.
structure GroupHom[G: Group, H: Group] {
    /// The mapping for the homomorphism.
    hom: G -> H
} constraint {
    is_group_hom(hom)
}

/// Group homomorphism extensionality: two homomorphisms are equal when they agree on every input.
theorem group_hom_ext[G: Group, H: Group](f: GroupHom[G, H], g: GroupHom[G, H]) {
    (forall(a: G) { f.hom(a) = g.hom(a) }) implies f = g
} by {
    if forall(a: G) { f.hom(a) = g.hom(a) } {
        f.hom = g.hom
    }
}

attributes GroupHom[G: Group, H: Group] {
    /// Group homomorphism extensionality from pointwise equality of the homomorphism.
    let ext = group_hom_ext[G, H]
}

/// Equal group homomorphisms have equal underlying functions.
theorem group_hom_eq_hom[G: Group, H: Group](f: GroupHom[G, H], g: GroupHom[G, H]) {
    f = g implies f.hom = g.hom
}

/// Equal group homomorphisms have equal values at every element.
theorem group_hom_eq_apply[G: Group, H: Group](f: GroupHom[G, H], g: GroupHom[G, H], a: G) {
    f = g implies f.hom(a) = g.hom(a)
} by {
    if f = g {
        f.hom(a) = g.hom(a)
    }
}

/// Equality of group homomorphisms transports predicates on group homomorphisms.
theorem group_hom_eq_transport_predicate[G: Group, H: Group](
    p: GroupHom[G, H] -> Bool,
    f: GroupHom[G, H],
    g: GroupHom[G, H]
) {
    f = g and p(f) implies p(g)
}

/// Equality of group homomorphisms transports predicates on group homomorphisms in the reverse direction.
theorem group_hom_eq_transport_predicate_rev[G: Group, H: Group](
    p: GroupHom[G, H] -> Bool,
    f: GroupHom[G, H],
    g: GroupHom[G, H]
) {
    f = g and p(g) implies p(f)
}

/// Equality of underlying functions determines equality of group homomorphisms.
theorem group_hom_eq_of_hom_eq[G: Group, H: Group](f: GroupHom[G, H], g: GroupHom[G, H]) {
    f.hom = g.hom implies f = g
} by {
    if f.hom = g.hom {
        forall(a: G) {
            f.hom(a) = g.hom(a)
        }
        group_hom_ext(f, g)
    }
}

/// Pointwise equality determines equality of group homomorphisms.
theorem group_hom_eq_of_apply_eq[G: Group, H: Group](f: GroupHom[G, H], g: GroupHom[G, H]) {
    (forall(a: G) { f.hom(a) = g.hom(a) }) implies f = g
} by {
    if forall(a: G) { f.hom(a) = g.hom(a) } {
        group_hom_ext(f, g)
    }
}

/// A group homomorphism preserves the operation.
theorem group_hom_mul[G: Group, H: Group](f: GroupHom[G, H], a: G, b: G) {
    f.hom(a * b) = f.hom(a) * f.hom(b)
} by {
    is_group_hom(f.hom)
}

/// A group homomorphism preserves the identity element.
theorem group_hom_one[G: Group, H: Group](f: GroupHom[G, H]) {
    f.hom(G.1) = H.1
} by {
    group_hom_mul(f, G.1, G.1)
    left_cancel(f.hom(G.1), f.hom(G.1), H.1)
}

/// A group homomorphism preserves inverses.
theorem group_hom_inv[G: Group, H: Group](f: GroupHom[G, H], a: G) {
    f.hom(a.inverse) = f.hom(a).inverse
} by {
    group_hom_mul(f, a, a.inverse)
    group_hom_one(f)
    left_cancel(f.hom(a), f.hom(a.inverse), f.hom(a).inverse)
}

/// A group homomorphism preserves powers.
theorem group_hom_pow[G: Group, H: Group](f: GroupHom[G, H], a: G, n: Nat) {
    f.hom(a.pow(n)) = f.hom(a).pow(n)
} by {
    define p(k: Nat) -> Bool {
        f.hom(a.pow(k)) = f.hom(a).pow(k)
    }

    a.pow(Nat.0) = G.1
    group_hom_one(f)
    f.hom(G.1) = H.1
    f.hom(a).pow(Nat.0) = H.1
    p(Nat.0)

    forall(k: Nat) {
        if p(k) {
            group_hom_mul(f, a, a.pow(k))
            f.hom(a) * f.hom(a.pow(k)) = f.hom(a) * f.hom(a).pow(k)
            p(k.suc)
        }
    }
}

/// A monoid homomorphism between groups is a group homomorphism.
theorem monoid_hom_imp_group_hom[G: Group, H: Group](f: G -> H) {
    is_monoid_hom(f) implies is_group_hom(f)
} by {
    if is_monoid_hom(f) {
        is_monoid_hom(f) = (f(G.1) = H.1 and forall(a: G, b: G) {
            f(a * b) = f(a) * f(b)
        })
        forall(a: G, b: G) {
            f(a * b) = f(a) * f(b)
        }
        is_group_hom(f)
    }
}

/// A group homomorphism preserves the full monoid structure.
theorem group_hom_imp_monoid_hom[G: Group, H: Group](f: G -> H) {
    is_group_hom(f) implies is_monoid_hom(f)
} by {
    if is_group_hom(f) {
        G.1 * G.1 = G.1
        f(G.1 * G.1) = f(G.1)
        f(G.1 * G.1) = f(G.1) * f(G.1)
        f(G.1) * f(G.1) = f(G.1)
        f(G.1) * f(G.1) = f(G.1) * H.1
        left_cancel(f(G.1), f(G.1), H.1)
        f(G.1) = H.1
        forall(a: G, b: G) {
            f(a * b) = f(a) * f(b)
        }
        is_monoid_hom(f)
    }
}

/// A bundled group homomorphism is a monoid homomorphism.
theorem group_hom_is_monoid_hom[G: Group, H: Group](f: GroupHom[G, H]) {
    is_monoid_hom(f.hom)
} by {
    group_hom_imp_monoid_hom(f.hom)
}

/// A bundled monoid homomorphism between groups is a group homomorphism.
theorem monoid_hom_is_group_hom[G: Group, H: Group](f: MonoidHom[G, H]) {
    is_group_hom(f.hom)
} by {
    monoid_hom_imp_group_hom(f.hom)
}

/// The monoid homomorphism underlying a group homomorphism.
let group_hom_to_monoid_hom[G: Group, H: Group](f: GroupHom[G, H]) -> result: MonoidHom[G, H] satisfy {
    MonoidHom.new(f.hom) = Option.some(result)
} by {
    group_hom_is_monoid_hom(f)
}

/// The underlying function of the monoid homomorphism associated to a group homomorphism.
theorem group_hom_to_monoid_hom_hom[G: Group, H: Group](f: GroupHom[G, H]) {
    group_hom_to_monoid_hom(f).hom = f.hom
}

/// The group homomorphism associated to a monoid homomorphism between groups.
let group_hom_of_monoid_hom[G: Group, H: Group](f: MonoidHom[G, H]) -> result: GroupHom[G, H] satisfy {
    GroupHom.new(f.hom) = Option.some(result)
} by {
    monoid_hom_is_group_hom(f)
}

/// The underlying function of the group homomorphism associated to a monoid homomorphism.
theorem group_hom_of_monoid_hom_hom[G: Group, H: Group](f: MonoidHom[G, H]) {
    group_hom_of_monoid_hom(f).hom = f.hom
}

/// Equality of functions transports the property of being a group homomorphism.
theorem function_eq_transport_group_hom[G: Group, H: Group](f: G -> H, g: G -> H) {
    f = g and is_group_hom(f) implies is_group_hom(g)
} by {
    if f = g and is_group_hom(f) {
        function_eq_transport_predicate(is_group_hom[G, H], f, g)
    }
}

/// Equality of functions transports the property of being a group homomorphism in the reverse direction.
theorem function_eq_transport_group_hom_rev[G: Group, H: Group](f: G -> H, g: G -> H) {
    f = g and is_group_hom(g) implies is_group_hom(f)
} by {
    if f = g and is_group_hom(g) {
        function_eq_transport_predicate_rev(is_group_hom[G, H], f, g)
    }
}

/// The identity function on a group preserves the group operation.
theorem identity_fn_is_group_hom[G: Group] {
    is_group_hom(identity_fn[G])
} by {
    identity_fn_is_monoid_hom[G]
    monoid_hom_imp_group_hom(identity_fn[G])
}

/// The composition of two group homomorphisms preserves the group operation.
theorem compose_is_group_hom[G: Group, H: Group, K: Group](f: H -> K, g: G -> H) {
    is_group_hom(f) and is_group_hom(g) implies is_group_hom(compose(f, g))
} by {
    if is_group_hom(f) and is_group_hom(g) {
        group_hom_imp_monoid_hom(f)
        group_hom_imp_monoid_hom(g)
        compose_is_monoid_hom(f, g)
        monoid_hom_imp_group_hom(compose(f, g))
        is_group_hom(compose(f, g))
    }
}

/// The identity homomorphism on a group, which sends every element to itself.
let identity_group_hom[G: Group]: GroupHom[G, G] satisfy {
    GroupHom.new(identity_fn[G]) = Option.some(identity_group_hom)
}

/// The underlying function of the identity homomorphism is the identity function.
theorem identity_group_hom_hom[G: Group] {
    identity_group_hom[G].hom = identity_fn[G]
}

/// The composition of two group homomorphisms.
let compose_group_hom[G: Group, H: Group, K: Group](f: GroupHom[H, K], g: GroupHom[G, H]) -> result: GroupHom[G, K] satisfy {
    GroupHom.new(compose(f.hom, g.hom)) = Option.some(result)
} by {
    compose_is_group_hom(f.hom, g.hom)
    is_group_hom(compose(f.hom, g.hom))
}

/// The underlying function of a composition of homomorphisms is the composition of underlying functions.
theorem compose_group_hom_hom[G: Group, H: Group, K: Group](f: GroupHom[H, K], g: GroupHom[G, H]) {
    compose_group_hom(f, g).hom = compose(f.hom, g.hom)
}

/// Composition of group homomorphisms is associative.
theorem compose_group_hom_assoc[A: Group, B: Group, C: Group, D: Group](
    f: GroupHom[C, D], g: GroupHom[B, C], h: GroupHom[A, B]) {
    compose_group_hom(compose_group_hom(f, g), h) = compose_group_hom(f, compose_group_hom(g, h))
} by {
    let lhs = compose_group_hom(compose_group_hom(f, g), h)
    let rhs = compose_group_hom(f, compose_group_hom(g, h))
    lhs.hom = compose(compose_group_hom(f, g).hom, h.hom)
    compose_group_hom(f, g).hom = compose(f.hom, g.hom)
    lhs.hom = compose(compose(f.hom, g.hom), h.hom)
    rhs.hom = compose(f.hom, compose_group_hom(g, h).hom)
    compose_group_hom(g, h).hom = compose(g.hom, h.hom)
    rhs.hom = compose(f.hom, compose(g.hom, h.hom))
    compose_assoc(f.hom, g.hom, h.hom)
    lhs.hom = rhs.hom
}

/// Composing the identity homomorphism on the left does nothing.
theorem compose_group_hom_identity_left[G: Group, H: Group](f: GroupHom[G, H]) {
    compose_group_hom(identity_group_hom[H], f) = f
} by {
    let lhs = compose_group_hom(identity_group_hom[H], f)
    lhs.hom = compose(identity_group_hom[H].hom, f.hom)
    identity_group_hom[H].hom = identity_fn[H]
    lhs.hom = compose(identity_fn[H], f.hom)
    compose_identity_left(f.hom)
    lhs.hom = f.hom
}

/// Composing the identity homomorphism on the right does nothing.
theorem compose_group_hom_identity_right[G: Group, H: Group](f: GroupHom[G, H]) {
    compose_group_hom(f, identity_group_hom[G]) = f
} by {
    let lhs = compose_group_hom(f, identity_group_hom[G])
    lhs.hom = compose(f.hom, identity_group_hom[G].hom)
    identity_group_hom[G].hom = identity_fn[G]
    lhs.hom = compose(f.hom, identity_fn[G])
    compose_identity_right(f.hom)
    lhs.hom = f.hom
}
