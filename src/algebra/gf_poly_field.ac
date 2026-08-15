/// Finite fields GF(p^n) as quotients of polynomial rings: the general
/// construction F[x]/(f) over a field F.
///
/// The base field is an arbitrary `Field` typeclass instance (the library's
/// prime fields Z/pZ are developed in `number_theory/finite_field.ac`, and
/// any such field can be plugged in here); the quotient is built from the
/// wrapped polynomial ring `PolyC[F]` (algebraic_geometry.ac), which carries
/// the ring typeclass instances that the raw `Polynomial` type lacks, and
/// the generic quotient-ring machinery `IdealQuotient` (algebra/ring/
/// quotient_ring.ac).
///
/// Contents:
///   - The principal ideal `(f)` of `F[x]` and the quotient ring
///     `F[x]/(f)` as a type with a `CommRing` instance.
///   - Equality in the quotient: `[a] = [b]` iff `a - b` lies in `(f)`;
///     `[a] = [0]` iff `f` divides `a`.
///   - The quotient is a field when `(f)` is a maximal ideal: every nonzero
///     class has a multiplicative inverse (`gf_quotient_maximal_inverse`),
///     and `[0] != [1]` (`gf_quotient_maximal_zero_ne_one`).  This is one
///     half of the classical equivalence
///         F[x]/(f) is a field  <==>  (f) is maximal  <==>  f irreducible.
///   - Irreducibility and primality for polynomials, and the bridge
///     "f prime (Bezout property) ==> the quotient is a field"
///     (`gf_quotient_field_inverse_of_prime`).
///
/// Recorded as next steps (see the comments at the end): the reverse
/// direction "quotient field ==> (f) maximal" (whose containment argument
/// needs the ideal-subset machinery), the classical equivalence "f
/// irreducible <==> f prime in F[x]" (Euclid's lemma for polynomials, which
/// needs the polynomial division algorithm and gcd/Bezout theory, not yet
/// in the library), and the counting theorem "F[x]/(f) has exactly p^n
/// elements when deg f = n and |F| = p" (needs the division algorithm to
/// reduce every class to a representative of degree < n, plus the counting
/// of degree-<n polynomials).
///
/// The concrete instances GF(4) = F_2[x]/(x^2+x+1) and
/// GF(8) = F_2[x]/(x^3+x+1), realized as enumerated fields with full
/// `Field` instances, live in `gf4.ac` and `gf8.ac`.

from nat import Nat
from comm_ring import CommRing
from algebra.field.field import Field
from algebra.algebraic_geometry import PolyC, polyc_one, polyc_zero, polyc_add, polyc_mul, polyc_neg,
    polyc_mul_comm_semigroup_law
from algebra.add_group_deep import add_group_sub_zero
from algebra.ring.ideal import Ideal, bundled_principal_ideal, is_maximal_ideal, is_maximal_ideal_proper,
    ideal_sum, principal_ideal, maximal_ideal_sum_principal_contains_one, ideal_contains_neg,
    ideal_contains_mul_left, bundled_principal_ideal_contains_eq, principal_ideal_contains_generator
from algebra.ring.quotient_ring import IdealQuotient, ideal_quotient_mk_type, ideal_quotient_type_mul,
    ideal_quotient_type_add, ideal_quotient_type_one, ideal_quotient_type_zero,
    ideal_quotient_type_mul_mk, ideal_quotient_type_add_mk, ideal_quotient_type_zero_mk,
    ideal_quotient_type_one_mk, ideal_quotient_type_zero_add, ideal_quotient_type_ext,
    ideal_quotient_mk_type_val
from algebra.ring.ideal_quotient import ideal_quotient_project_eq_iff_contains_sub,
    ideal_quotient_project

numerals Nat

/// The principal ideal of `F[x]` generated by the wrapped polynomial `f`.
define gf_ideal[F: Field](f: PolyC[F]) -> Ideal[PolyC[F]] {
    bundled_principal_ideal[PolyC[F]](f)
}

/// Divisibility in the wrapped polynomial ring `F[x]`: `g` is a multiple of
/// `f` when some `q` satisfies `f * q = g`.
define polyc_divides[F: CommRing](f: PolyC[F], g: PolyC[F]) -> Bool {
    exists(q: PolyC[F]) {
        polyc_mul(f, q) = g
    }
}

/// A polynomial is a unit when it divides one, i.e. it has a multiplicative
/// inverse in the polynomial ring.
define is_unit_poly[F: CommRing](p: PolyC[F]) -> Bool {
    exists(q: PolyC[F]) {
        polyc_mul(p, q) = polyc_one[F]
    }
}

/// Irreducibility of a polynomial: it is nonzero, not a unit, and has no
/// factorization into two non-units.
define is_irreducible_poly[F: Field](f: PolyC[F]) -> Bool {
    f != polyc_zero[F]
    and not is_unit_poly(f)
    and not exists(g: PolyC[F], h: PolyC[F]) {
        polyc_mul(g, h) = f and not is_unit_poly(g) and not is_unit_poly(h)
    }
}

/// Primality of a polynomial in Bezout form: every polynomial not divisible
/// by `f` is coprime to `f` in the sense that `a * f + b * g = 1` for some
/// `a`, `b`.  Over a field, this is equivalent to irreducibility (Euclid's
/// lemma for `F[x]`, recorded at the end); the Bezout form is what the
/// quotient field property uses directly.
define is_prime_poly[F: Field](f: PolyC[F]) -> Bool {
    f != polyc_zero[F]
    and forall(g: PolyC[F]) {
        not polyc_divides(f, g) implies
        exists(a: PolyC[F], b: PolyC[F]) {
            polyc_add(polyc_mul(a, f), polyc_mul(b, g)) = polyc_one[F]
        }
    }
}

/// The apply lemma for primality: a polynomial not divisible by a prime
/// polynomial `f` has a Bezout partner `a * f + b * g = 1`.
theorem is_prime_poly_apply[F: Field](f: PolyC[F], g: PolyC[F]) {
    is_prime_poly(f) and not polyc_divides(f, g) implies
    exists(a: PolyC[F], b: PolyC[F]) {
        polyc_add(polyc_mul(a, f), polyc_mul(b, g)) = polyc_one[F]
    }
} by {
    if is_prime_poly(f) and not polyc_divides(f, g) {
        is_prime_poly(f) = (f != polyc_zero[F] and forall(g0: PolyC[F]) {
            not polyc_divides(f, g0) implies
            exists(a: PolyC[F], b: PolyC[F]) {
                polyc_add(polyc_mul(a, f), polyc_mul(b, g0)) = polyc_one[F]
            }
        })
        if not polyc_divides(f, g) {
            exists(a: PolyC[F], b: PolyC[F]) {
                polyc_add(polyc_mul(a, f), polyc_mul(b, g)) = polyc_one[F]
            }
        }
    }
}

// ---------------------------------------------------------------------------
// Equality in the quotient ring.
// ---------------------------------------------------------------------------

/// Equality of quotient classes is membership of the difference in the
/// ideal: `[a] = [b]` iff `a - b` lies in `(f)`.
theorem gf_quotient_mk_eq_iff_contains_sub[F: Field](f: PolyC[F], a: PolyC[F], b: PolyC[F]) {
    (ideal_quotient_mk_type(gf_ideal(f), a) = ideal_quotient_mk_type(gf_ideal(f), b)) =
        gf_ideal(f).contains(a - b)
} by {
    if ideal_quotient_mk_type(gf_ideal(f), a) = ideal_quotient_mk_type(gf_ideal(f), b) {
        ideal_quotient_mk_type(gf_ideal(f), a).val = ideal_quotient_mk_type(gf_ideal(f), b).val
        ideal_quotient_mk_type_val(gf_ideal(f), a)
        ideal_quotient_mk_type_val(gf_ideal(f), b)
        ideal_quotient_project(gf_ideal(f), a) = ideal_quotient_project(gf_ideal(f), b)
        ideal_quotient_project_eq_iff_contains_sub(gf_ideal(f), a, b)
        gf_ideal(f).contains(a - b)
    }
    if gf_ideal(f).contains(a - b) {
        ideal_quotient_project_eq_iff_contains_sub(gf_ideal(f), a, b)
        ideal_quotient_project(gf_ideal(f), a) = ideal_quotient_project(gf_ideal(f), b)
        ideal_quotient_mk_type_val(gf_ideal(f), a)
        ideal_quotient_mk_type_val(gf_ideal(f), b)
        ideal_quotient_mk_type(gf_ideal(f), a).val = ideal_quotient_mk_type(gf_ideal(f), b).val
        ideal_quotient_type_ext(gf_ideal(f), ideal_quotient_mk_type(gf_ideal(f), a),
            ideal_quotient_mk_type(gf_ideal(f), b))
        ideal_quotient_mk_type(gf_ideal(f), a) = ideal_quotient_mk_type(gf_ideal(f), b)
    }
    (ideal_quotient_mk_type(gf_ideal(f), a) = ideal_quotient_mk_type(gf_ideal(f), b)) =
        gf_ideal(f).contains(a - b)
}

/// A class is zero exactly when its representative lies in the ideal:
/// `[a] = [0]` iff `f` divides `a`.
theorem gf_quotient_mk_eq_zero_iff_contains[F: Field](f: PolyC[F], a: PolyC[F]) {
    (ideal_quotient_mk_type(gf_ideal(f), a) = ideal_quotient_type_zero(gf_ideal(f))) =
        gf_ideal(f).contains(a)
} by {
    ideal_quotient_type_zero_mk(gf_ideal(f))
    ideal_quotient_type_zero(gf_ideal(f)) = ideal_quotient_mk_type(gf_ideal(f), PolyC[F].zero)
    gf_quotient_mk_eq_iff_contains_sub(f, a, PolyC[F].zero)
    (ideal_quotient_mk_type(gf_ideal(f), a) = ideal_quotient_mk_type(gf_ideal(f), PolyC[F].zero)) =
        gf_ideal(f).contains(a - PolyC[F].zero)
    add_group_sub_zero(a)
    a - PolyC[F].zero = a
    gf_ideal(f).contains(a - PolyC[F].zero) = gf_ideal(f).contains(a)
    (ideal_quotient_mk_type(gf_ideal(f), a) = ideal_quotient_mk_type(gf_ideal(f), PolyC[F].zero)) =
        gf_ideal(f).contains(a)
    (ideal_quotient_mk_type(gf_ideal(f), a) = ideal_quotient_type_zero(gf_ideal(f))) =
        gf_ideal(f).contains(a)
}

// ---------------------------------------------------------------------------
// The quotient is a field when (f) is maximal.
// ---------------------------------------------------------------------------

/// If `(f)` is a maximal ideal then every nonzero class of `F[x]/(f)` has a
/// multiplicative inverse: for `a` outside `(f)` some `m` satisfies
/// `[a] * [m] = [1]`.  This is the field axiom of the quotient ring.
///
/// Proof sketch: maximality of `(f)` together with `a` outside `(f)` yields
/// `x + m * a = 1` with `x` in `(f)` (`maximal_ideal_sum_principal_
/// contains_one`); then `[x] = [0]`, so `[1] = [x] + [m * a] = [0] + [m * a]
/// = [m * a] = [a] * [m]`.
theorem gf_quotient_maximal_inverse[F: Field](f: PolyC[F], a: PolyC[F]) {
    is_maximal_ideal(gf_ideal(f).contains) and not gf_ideal(f).contains(a)
    implies exists(m: PolyC[F]) {
        ideal_quotient_type_mul(gf_ideal(f),
            ideal_quotient_mk_type(gf_ideal(f), a),
            ideal_quotient_mk_type(gf_ideal(f), m)) =
        ideal_quotient_type_one(gf_ideal(f))
    }
} by {
    if is_maximal_ideal(gf_ideal(f).contains) and not gf_ideal(f).contains(a) {
        maximal_ideal_sum_principal_contains_one[PolyC[F]](gf_ideal(f).contains, a)
        ideal_sum[PolyC[F]](gf_ideal(f).contains, principal_ideal[PolyC[F]](a), PolyC[F].one)
        ideal_sum[PolyC[F]](gf_ideal(f).contains, principal_ideal[PolyC[F]](a), PolyC[F].one) =
            exists(k0: PolyC[F], k1: PolyC[F]) {
                gf_ideal(f).contains(k0) and principal_ideal[PolyC[F]](a, k1) and
                PolyC[F].one = k0 + k1
            }
        exists(k0: PolyC[F], k1: PolyC[F]) {
            gf_ideal(f).contains(k0) and principal_ideal[PolyC[F]](a, k1) and
            PolyC[F].one = k0 + k1
        }
        let (x: PolyC[F], y: PolyC[F]) satisfy {
            gf_ideal(f).contains(x) and
            principal_ideal[PolyC[F]](a, y) and
            PolyC[F].one = x + y
        }
        principal_ideal[PolyC[F]](a, y) = exists(r: PolyC[F]) {
            y = r * a
        }
        exists(r: PolyC[F]) {
            y = r * a
        }
        let (m: PolyC[F]) satisfy {
            y = m * a
        }
        let h = m * a
        PolyC[F].one = x + h
        // [x] = [0] because x lies in (f).
        gf_quotient_mk_eq_zero_iff_contains(f, x)
        ideal_quotient_mk_type(gf_ideal(f), x) = ideal_quotient_type_zero(gf_ideal(f))
        // [x] + [h] = [x + h] = [one] = [1].
        ideal_quotient_type_add_mk(gf_ideal(f), x, h)
        ideal_quotient_type_add(gf_ideal(f), ideal_quotient_mk_type(gf_ideal(f), x),
            ideal_quotient_mk_type(gf_ideal(f), h)) =
            ideal_quotient_mk_type(gf_ideal(f), x + h)
        ideal_quotient_mk_type(gf_ideal(f), x + h) =
            ideal_quotient_mk_type(gf_ideal(f), PolyC[F].one)
        ideal_quotient_type_one_mk(gf_ideal(f))
        ideal_quotient_mk_type(gf_ideal(f), PolyC[F].one) =
            ideal_quotient_type_one(gf_ideal(f))
        ideal_quotient_type_add(gf_ideal(f), ideal_quotient_mk_type(gf_ideal(f), x),
            ideal_quotient_mk_type(gf_ideal(f), h)) =
            ideal_quotient_type_one(gf_ideal(f))
        // [0] + [h] = [h], so [h] = [1].
        ideal_quotient_type_zero_add(gf_ideal(f), ideal_quotient_mk_type(gf_ideal(f), h))
        ideal_quotient_type_add(gf_ideal(f), ideal_quotient_type_zero(gf_ideal(f)),
            ideal_quotient_mk_type(gf_ideal(f), h)) =
            ideal_quotient_mk_type(gf_ideal(f), h)
        ideal_quotient_mk_type(gf_ideal(f), h) = ideal_quotient_type_one(gf_ideal(f))
        // [a] * [m] = [a * m] = [m * a] = [h] = [1].
        ideal_quotient_type_mul_mk(gf_ideal(f), a, m)
        ideal_quotient_type_mul(gf_ideal(f), ideal_quotient_mk_type(gf_ideal(f), a),
            ideal_quotient_mk_type(gf_ideal(f), m)) =
            ideal_quotient_mk_type(gf_ideal(f), a * m)
        polyc_mul_comm_semigroup_law(a, m)
        a * m = m * a
        m * a = h
        a * m = h
        ideal_quotient_mk_type(gf_ideal(f), a * m) = ideal_quotient_mk_type(gf_ideal(f), h)
        ideal_quotient_type_mul(gf_ideal(f), ideal_quotient_mk_type(gf_ideal(f), a),
            ideal_quotient_mk_type(gf_ideal(f), m)) =
            ideal_quotient_type_one(gf_ideal(f))
        exists(m0: PolyC[F]) {
            ideal_quotient_type_mul(gf_ideal(f),
                ideal_quotient_mk_type(gf_ideal(f), a),
                ideal_quotient_mk_type(gf_ideal(f), m0)) =
            ideal_quotient_type_one(gf_ideal(f))
        }
    }
}

/// If `(f)` is a maximal ideal then the quotient `F[x]/(f)` has `[0] != [1]`:
/// a maximal ideal is proper, so one does not lie in `(f)`, while `[0] = [1]`
/// would force `1` into `(f)`.
theorem gf_quotient_maximal_zero_ne_one[F: Field](f: PolyC[F]) {
    is_maximal_ideal(gf_ideal(f).contains) implies
        ideal_quotient_type_zero(gf_ideal(f)) != ideal_quotient_type_one(gf_ideal(f))
} by {
    if is_maximal_ideal(gf_ideal(f).contains) {
        if ideal_quotient_type_zero(gf_ideal(f)) = ideal_quotient_type_one(gf_ideal(f)) {
            ideal_quotient_type_zero_mk(gf_ideal(f))
            ideal_quotient_type_one_mk(gf_ideal(f))
            ideal_quotient_mk_type(gf_ideal(f), PolyC[F].zero) =
                ideal_quotient_mk_type(gf_ideal(f), PolyC[F].one)
            gf_quotient_mk_eq_iff_contains_sub(f, PolyC[F].one, PolyC[F].zero)
            gf_ideal(f).contains(PolyC[F].one - PolyC[F].zero)
            is_maximal_ideal_proper(gf_ideal(f).contains)
            not gf_ideal(f).contains(PolyC[F].one)
            PolyC[F].one - PolyC[F].zero = PolyC[F].one
            gf_ideal(f).contains(PolyC[F].one)
            false
        }
        ideal_quotient_type_zero(gf_ideal(f)) != ideal_quotient_type_one(gf_ideal(f))
    }
}

// ---------------------------------------------------------------------------
// Irreducibility, primality, and the field property.
// ---------------------------------------------------------------------------

/// If `f` is prime in the Bezout sense then the quotient `F[x]/(f)` is a
/// field: every class not represented by a multiple of `f` has an inverse.
/// This is the statement-level form of "F[x]/(f) is a field for f prime".
///
/// Proof sketch: primality gives `a * f + b * g = 1`; then
/// `[g] * [b] = [g * b] = [b * g] = [1]` because `b * g - 1 = -(a * f)`
/// lies in `(f)`.
theorem gf_quotient_field_inverse_of_prime[F: Field](f: PolyC[F], g: PolyC[F]) {
    is_prime_poly(f) and not polyc_divides(f, g)
    implies exists(b: PolyC[F]) {
        ideal_quotient_type_mul(gf_ideal(f),
            ideal_quotient_mk_type(gf_ideal(f), g),
            ideal_quotient_mk_type(gf_ideal(f), b)) =
        ideal_quotient_type_one(gf_ideal(f))
    }
} by {
    if is_prime_poly(f) and not polyc_divides(f, g) {
        is_prime_poly_apply(f, g)
        exists(a: PolyC[F], b: PolyC[F]) {
            polyc_add(polyc_mul(a, f), polyc_mul(b, g)) = polyc_one[F]
        }
        let (a: PolyC[F], b: PolyC[F]) satisfy {
            polyc_add(polyc_mul(a, f), polyc_mul(b, g)) = polyc_one[F]
        }
        let h = polyc_mul(b, g)
        polyc_mul(a, f) + h = polyc_one[F]
        // a * f lies in (f), so [a * f] = [0].
        bundled_principal_ideal_contains_eq[PolyC[F]](f, polyc_mul(a, f))
        gf_ideal(f).contains(polyc_mul(a, f)) = principal_ideal[PolyC[F]](f, polyc_mul(a, f))
        principal_ideal[PolyC[F]](f, polyc_mul(a, f)) = exists(r: PolyC[F]) {
            polyc_mul(a, f) = r * f
        }
        polyc_mul(a, f) = a * f
        exists(r: PolyC[F]) {
            polyc_mul(a, f) = r * f
        }
        principal_ideal[PolyC[F]](f, polyc_mul(a, f))
        gf_ideal(f).contains(polyc_mul(a, f))
        gf_quotient_mk_eq_zero_iff_contains(f, polyc_mul(a, f))
        ideal_quotient_mk_type(gf_ideal(f), polyc_mul(a, f)) =
            ideal_quotient_type_zero(gf_ideal(f))
        // [a * f] + [h] = [a * f + h] = [one] = [1].
        ideal_quotient_type_add_mk(gf_ideal(f), polyc_mul(a, f), h)
        ideal_quotient_type_add(gf_ideal(f), ideal_quotient_mk_type(gf_ideal(f), polyc_mul(a, f)),
            ideal_quotient_mk_type(gf_ideal(f), h)) =
            ideal_quotient_mk_type(gf_ideal(f), polyc_mul(a, f) + h)
        ideal_quotient_mk_type(gf_ideal(f), polyc_mul(a, f) + h) =
            ideal_quotient_mk_type(gf_ideal(f), polyc_one[F])
        ideal_quotient_type_one_mk(gf_ideal(f))
        ideal_quotient_mk_type(gf_ideal(f), polyc_one[F]) =
            ideal_quotient_type_one(gf_ideal(f))
        ideal_quotient_type_add(gf_ideal(f), ideal_quotient_mk_type(gf_ideal(f), polyc_mul(a, f)),
            ideal_quotient_mk_type(gf_ideal(f), h)) =
            ideal_quotient_type_one(gf_ideal(f))
        // [0] + [h] = [h], so [h] = [1].
        ideal_quotient_type_zero_add(gf_ideal(f), ideal_quotient_mk_type(gf_ideal(f), h))
        ideal_quotient_type_add(gf_ideal(f), ideal_quotient_type_zero(gf_ideal(f)),
            ideal_quotient_mk_type(gf_ideal(f), h)) =
            ideal_quotient_mk_type(gf_ideal(f), h)
        ideal_quotient_mk_type(gf_ideal(f), h) = ideal_quotient_type_one(gf_ideal(f))
        // [g] * [b] = [g * b] = [b * g] = [h] = [1].
        ideal_quotient_type_mul_mk(gf_ideal(f), g, b)
        ideal_quotient_type_mul(gf_ideal(f), ideal_quotient_mk_type(gf_ideal(f), g),
            ideal_quotient_mk_type(gf_ideal(f), b)) =
            ideal_quotient_mk_type(gf_ideal(f), g * b)
        polyc_mul_comm_semigroup_law(g, b)
        g * b = polyc_mul(b, g)
        polyc_mul(b, g) = h
        g * b = h
        ideal_quotient_mk_type(gf_ideal(f), g * b) = ideal_quotient_mk_type(gf_ideal(f), h)
        ideal_quotient_type_mul(gf_ideal(f), ideal_quotient_mk_type(gf_ideal(f), g),
            ideal_quotient_mk_type(gf_ideal(f), b)) =
            ideal_quotient_type_one(gf_ideal(f))
        exists(b0: PolyC[F]) {
            ideal_quotient_type_mul(gf_ideal(f),
                ideal_quotient_mk_type(gf_ideal(f), g),
                ideal_quotient_mk_type(gf_ideal(f), b0)) =
            ideal_quotient_type_one(gf_ideal(f))
        }
    }
}

// ---------------------------------------------------------------------------
// Recorded next steps (gaps).
// ---------------------------------------------------------------------------

// The classical equivalence chain for a field F and a polynomial f:
//
//    F[x]/(f) is a field  <==>  (f) is a maximal ideal  <==>  f is irreducible
//
// The forward direction "maximal ==> field" is proved above
// (`gf_quotient_maximal_inverse`, `gf_quotient_maximal_zero_ne_one`).  The
// reverse direction "field ==> maximal" goes as follows: if J is a proper
// ideal with (f) contained in J and a is an element of J outside (f), then
// the class [a] is nonzero, so by the field property [a] * [m] = [1] for
// some m; hence a * m - 1 lies in (f) (contained in J), and a * m lies in
// J, forcing 1 into J, contradicting properness.  Formalizing this needs
// the ideal-subset machinery (`is_maximal_ideal_from_constraints` and
// ideal containment reasoning); it is recorded here rather than proved.
//
// The equivalence "(f) maximal <==> f irreducible" needs Euclid's lemma for
// polynomials over a field -- "an irreducible polynomial divides a product
// only if it divides a factor", equivalently "irreducible <==> prime" --
// which in turn needs the polynomial division algorithm (every polynomial
// is a multiple of a divisor plus a remainder of smaller degree) and the
// gcd/Bezout theory for F[x]; none of these are in the library yet.  Until
// then the field property for irreducible f is exposed through
// `is_prime_poly` and `gf_quotient_field_inverse_of_prime`.
//
// The counting theorem -- F[x]/(f) has exactly p^n elements when f has
// degree n and the base field has p elements -- likewise needs the division
// algorithm (to represent every class by the unique polynomial of degree
// < n) together with the combinatorial count of degree-<n polynomials:
//
//   theorem gf_quotient_card[F: Field](f: PolyC[F], n: Nat, p: Nat) {
//       polynomial_degree_exact(f, n) and ... implies
//       card(gf_quotient(f)) = p.pow(n)
//   }
//
// For the concrete fields GF(4) and GF(8) the element counts are proved
// directly by enumeration in gf4.ac and gf8.ac: GF(4) has the four elements
// [0], [1], [X], [X + 1] and GF(8) the eight elements [0], [1], [X],
// [X + 1], [X^2], [X^2 + 1], [X^2 + X], [X^2 + X + 1].
//
// The multiplicative group of GF(p^n) is cyclic of order p^n - 1.  This
// needs finite-group cyclicity theory (the library's cyclic-order bridge,
// `finite_group/cyclic_order_bridge.ac`, and the primitive-root theory of
// `number_theory/primitive_root*.ac`, which currently only assume the
// existence of a generator of Z/pZ*) and is recorded here as the next step.
