from pair import Pair, pair_ext, pair_map
from algebra.add import Add
from algebra.add_semigroup import AddSemigroup
from algebra.add_comm_semigroup import AddCommSemigroup
from algebra.add_monoid import AddMonoid
from algebra.add_group import AddGroup
from algebra.mul import Mul
from algebra.semigroup import Semigroup
from algebra.comm_semigroup import CommSemigroup
from algebra.one import One
from algebra.zero import Zero
from algebra.neg import Neg
from algebra.monoid.monoid import Monoid
from algebra.group import Group, inverse_inverse, inverse_mul
from semiring import Semiring

/// Componentwise addition on a product.
define pair_add[A: Add, B: Add](p: Pair[A, B], q: Pair[A, B]) -> Pair[A, B] {
    Pair.new(p.first + q.first, p.second + q.second)
}

/// Componentwise multiplication on a product.
define pair_mul[A: Mul, B: Mul](p: Pair[A, B], q: Pair[A, B]) -> Pair[A, B] {
    Pair.new(p.first * q.first, p.second * q.second)
}

/// Componentwise zero on a product.
let pair_zero[A: Zero, B: Zero]: Pair[A, B] =
    Pair.new(A.0, B.0)

/// Componentwise one on a product.
let pair_one[A: One, B: One]: Pair[A, B] =
    Pair.new(A.1, B.1)

/// Componentwise negation on a product.
define pair_neg[A: Neg, B: Neg](p: Pair[A, B]) -> Pair[A, B] {
    Pair.new(-p.first, -p.second)
}

/// Componentwise inversion on a product.
define pair_inverse[A: Group, B: Group](p: Pair[A, B]) -> Pair[A, B] {
    Pair.new(p.first.inverse, p.second.inverse)
}

/// The first coordinate of a product sum.
theorem pair_add_first[A: Add, B: Add](p: Pair[A, B], q: Pair[A, B]) {
    pair_add(p, q).first = p.first + q.first
}

/// The second coordinate of a product sum.
theorem pair_add_second[A: Add, B: Add](p: Pair[A, B], q: Pair[A, B]) {
    pair_add(p, q).second = p.second + q.second
}

/// The first coordinate of a product product.
theorem pair_mul_first[A: Mul, B: Mul](p: Pair[A, B], q: Pair[A, B]) {
    pair_mul(p, q).first = p.first * q.first
}

/// The second coordinate of a product product.
theorem pair_mul_second[A: Mul, B: Mul](p: Pair[A, B], q: Pair[A, B]) {
    pair_mul(p, q).second = p.second * q.second
}

/// The first coordinate of the product zero.
theorem pair_zero_first[A: Zero, B: Zero] {
    pair_zero[A, B].first = A.0
}

/// The second coordinate of the product zero.
theorem pair_zero_second[A: Zero, B: Zero] {
    pair_zero[A, B].second = B.0
}

/// The first coordinate of the product one.
theorem pair_one_first[A: One, B: One] {
    pair_one[A, B].first = A.1
}

/// The second coordinate of the product one.
theorem pair_one_second[A: One, B: One] {
    pair_one[A, B].second = B.1
}

/// The first coordinate of a product negation.
theorem pair_neg_first[A: Neg, B: Neg](p: Pair[A, B]) {
    pair_neg(p).first = -p.first
}

/// The second coordinate of a product negation.
theorem pair_neg_second[A: Neg, B: Neg](p: Pair[A, B]) {
    pair_neg(p).second = -p.second
}

/// The first coordinate of a product inverse.
theorem pair_inverse_first[A: Group, B: Group](p: Pair[A, B]) {
    pair_inverse(p).first = p.first.inverse
}

/// The second coordinate of a product inverse.
theorem pair_inverse_second[A: Group, B: Group](p: Pair[A, B]) {
    pair_inverse(p).second = p.second.inverse
}

/// Componentwise addition is associative on a product of additive semigroups.
theorem pair_add_assoc[A: AddSemigroup, B: AddSemigroup](
    p: Pair[A, B], q: Pair[A, B], r: Pair[A, B]
) {
    pair_add(p, pair_add(q, r)) = pair_add(pair_add(p, q), r)
} by {
    let lhs = pair_add(p, pair_add(q, r))
    let rhs = pair_add(pair_add(p, q), r)
    lhs.first = p.first + (q.first + r.first)
    rhs.first = (p.first + q.first) + r.first
    lhs.first = rhs.first
    lhs.second = p.second + (q.second + r.second)
    rhs.second = (p.second + q.second) + r.second
    lhs.second = rhs.second
    pair_ext(lhs, rhs)
}

/// Componentwise addition is commutative on a product of commutative additive semigroups.
theorem pair_add_comm[A: AddCommSemigroup, B: AddCommSemigroup](p: Pair[A, B], q: Pair[A, B]) {
    pair_add(p, q) = pair_add(q, p)
} by {
    let lhs = pair_add(p, q)
    let rhs = pair_add(q, p)
    lhs.first = p.first + q.first
    rhs.first = q.first + p.first
    lhs.first = rhs.first
    lhs.second = p.second + q.second
    rhs.second = q.second + p.second
    lhs.second = rhs.second
    pair_ext(lhs, rhs)
}

/// Adding the componentwise zero on the right changes nothing.
theorem pair_add_zero_right[A: AddMonoid, B: AddMonoid](p: Pair[A, B]) {
    pair_add(p, pair_zero[A, B]) = p
} by {
    let lhs = pair_add(p, pair_zero[A, B])
    lhs.first = p.first + A.0
    lhs.first = p.first
    lhs.second = p.second + B.0
    lhs.second = p.second
    pair_ext(lhs, p)
}

/// Adding the componentwise zero on the left changes nothing.
theorem pair_add_zero_left[A: AddMonoid, B: AddMonoid](p: Pair[A, B]) {
    pair_add(pair_zero[A, B], p) = p
} by {
    let lhs = pair_add(pair_zero[A, B], p)
    lhs.first = A.0 + p.first
    lhs.first = p.first
    lhs.second = B.0 + p.second
    lhs.second = p.second
    pair_ext(lhs, p)
}

/// Componentwise negation is a right additive inverse.
theorem pair_add_neg_right[A: AddGroup, B: AddGroup](p: Pair[A, B]) {
    pair_add(p, pair_neg(p)) = pair_zero[A, B]
} by {
    let lhs = pair_add(p, pair_neg(p))
    lhs.first = p.first + -p.first
    lhs.first = A.0
    lhs.second = p.second + -p.second
    lhs.second = B.0
    pair_ext(lhs, pair_zero[A, B])
}

/// Componentwise negation is a left additive inverse.
theorem pair_add_neg_left[A: AddGroup, B: AddGroup](p: Pair[A, B]) {
    pair_add(pair_neg(p), p) = pair_zero[A, B]
} by {
    let lhs = pair_add(pair_neg(p), p)
    lhs.first = -p.first + p.first
    lhs.first = A.0
    lhs.second = -p.second + p.second
    lhs.second = B.0
    pair_ext(lhs, pair_zero[A, B])
}

/// Componentwise multiplication is associative on a product of semigroups.
theorem pair_mul_assoc[A: Semigroup, B: Semigroup](
    p: Pair[A, B], q: Pair[A, B], r: Pair[A, B]
) {
    pair_mul(p, pair_mul(q, r)) = pair_mul(pair_mul(p, q), r)
} by {
    let lhs = pair_mul(p, pair_mul(q, r))
    let rhs = pair_mul(pair_mul(p, q), r)
    lhs.first = p.first * (q.first * r.first)
    rhs.first = (p.first * q.first) * r.first
    lhs.first = rhs.first
    lhs.second = p.second * (q.second * r.second)
    rhs.second = (p.second * q.second) * r.second
    lhs.second = rhs.second
    pair_ext(lhs, rhs)
}

/// Componentwise multiplication is commutative on a product of commutative semigroups.
theorem pair_mul_comm[A: CommSemigroup, B: CommSemigroup](p: Pair[A, B], q: Pair[A, B]) {
    pair_mul(p, q) = pair_mul(q, p)
} by {
    let lhs = pair_mul(p, q)
    let rhs = pair_mul(q, p)
    lhs.first = p.first * q.first
    rhs.first = q.first * p.first
    lhs.first = rhs.first
    lhs.second = p.second * q.second
    rhs.second = q.second * p.second
    lhs.second = rhs.second
    pair_ext(lhs, rhs)
}

/// Multiplying by the componentwise one on the right changes nothing.
theorem pair_mul_one_right[A: Monoid, B: Monoid](p: Pair[A, B]) {
    pair_mul(p, pair_one[A, B]) = p
} by {
    let lhs = pair_mul(p, pair_one[A, B])
    lhs.first = p.first * A.1
    lhs.first = p.first
    lhs.second = p.second * B.1
    lhs.second = p.second
    pair_ext(lhs, p)
}

/// Multiplying by the componentwise one on the left changes nothing.
theorem pair_mul_one_left[A: Monoid, B: Monoid](p: Pair[A, B]) {
    pair_mul(pair_one[A, B], p) = p
} by {
    let lhs = pair_mul(pair_one[A, B], p)
    lhs.first = A.1 * p.first
    lhs.first = p.first
    lhs.second = B.1 * p.second
    lhs.second = p.second
    pair_ext(lhs, p)
}

/// Componentwise inversion is a right inverse.
theorem pair_mul_inverse_right[A: Group, B: Group](p: Pair[A, B]) {
    pair_mul(p, pair_inverse[A, B](p)) = pair_one[A, B]
} by {
    let lhs = pair_mul(p, pair_inverse[A, B](p))
    lhs.first = p.first * p.first.inverse
    lhs.first = A.1
    lhs.second = p.second * p.second.inverse
    lhs.second = B.1
    pair_ext(lhs, pair_one[A, B])
}

/// Componentwise inversion is a left inverse.
theorem pair_mul_inverse_left[A: Group, B: Group](p: Pair[A, B]) {
    pair_mul(pair_inverse[A, B](p), p) = pair_one[A, B]
} by {
    let lhs = pair_mul(pair_inverse[A, B](p), p)
    lhs.first = p.first.inverse * p.first
    lhs.first = A.1
    lhs.second = p.second.inverse * p.second
    lhs.second = B.1
    pair_ext(lhs, pair_one[A, B])
}

/// Componentwise maps preserve componentwise zero when both maps preserve zero.
theorem pair_map_pair_zero[A: Zero, B: Zero, C: Zero, D: Zero](f: A -> C, g: B -> D) {
    f(A.0) = C.0 and g(B.0) = D.0 implies
    pair_map(f, g, pair_zero[A, B]) = pair_zero[C, D]
} by {
    if f(A.0) = C.0 and g(B.0) = D.0 {
        let lhs = pair_map(f, g, pair_zero[A, B])
        lhs.first = f(pair_zero[A, B].first)
        pair_zero[A, B].first = A.0
        lhs.first = f(A.0)
        lhs.first = C.0
        lhs.second = g(pair_zero[A, B].second)
        pair_zero[A, B].second = B.0
        lhs.second = g(B.0)
        lhs.second = D.0
        pair_ext(lhs, pair_zero[C, D])
    }
}

/// Componentwise maps preserve componentwise one when both maps preserve one.
theorem pair_map_pair_one[A: One, B: One, C: One, D: One](f: A -> C, g: B -> D) {
    f(A.1) = C.1 and g(B.1) = D.1 implies
    pair_map(f, g, pair_one[A, B]) = pair_one[C, D]
} by {
    if f(A.1) = C.1 and g(B.1) = D.1 {
        let lhs = pair_map(f, g, pair_one[A, B])
        lhs.first = f(pair_one[A, B].first)
        pair_one[A, B].first = A.1
        lhs.first = f(A.1)
        lhs.first = C.1
        lhs.second = g(pair_one[A, B].second)
        pair_one[A, B].second = B.1
        lhs.second = g(B.1)
        lhs.second = D.1
        pair_ext(lhs, pair_one[C, D])
    }
}

/// Componentwise multiplication distributes over componentwise addition from the left.
theorem pair_mul_add_distrib_left[A: Semiring, B: Semiring](
    p: Pair[A, B],
    q: Pair[A, B],
    r: Pair[A, B]
) {
    pair_mul(p, pair_add(q, r)) = pair_add(pair_mul(p, q), pair_mul(p, r))
} by {
    let lhs = pair_mul(p, pair_add(q, r))
    let rhs = pair_add(pair_mul(p, q), pair_mul(p, r))
    lhs.first = p.first * pair_add(q, r).first
    pair_add(q, r).first = q.first + r.first
    lhs.first = p.first * (q.first + r.first)
    lhs.first = p.first * q.first + p.first * r.first
    rhs.first = pair_mul(p, q).first + pair_mul(p, r).first
    pair_mul(p, q).first = p.first * q.first
    pair_mul(p, r).first = p.first * r.first
    rhs.first = p.first * q.first + p.first * r.first
    lhs.first = rhs.first
    lhs.second = p.second * pair_add(q, r).second
    pair_add(q, r).second = q.second + r.second
    lhs.second = p.second * (q.second + r.second)
    lhs.second = p.second * q.second + p.second * r.second
    rhs.second = pair_mul(p, q).second + pair_mul(p, r).second
    pair_mul(p, q).second = p.second * q.second
    pair_mul(p, r).second = p.second * r.second
    rhs.second = p.second * q.second + p.second * r.second
    lhs.second = rhs.second
    pair_ext(lhs, rhs)
}

/// Componentwise multiplication distributes over componentwise addition from the right.
theorem pair_mul_add_distrib_right[A: Semiring, B: Semiring](
    p: Pair[A, B],
    q: Pair[A, B],
    r: Pair[A, B]
) {
    pair_mul(pair_add(p, q), r) = pair_add(pair_mul(p, r), pair_mul(q, r))
} by {
    let lhs = pair_mul(pair_add(p, q), r)
    let rhs = pair_add(pair_mul(p, r), pair_mul(q, r))
    lhs.first = pair_add(p, q).first * r.first
    pair_add(p, q).first = p.first + q.first
    lhs.first = (p.first + q.first) * r.first
    lhs.first = p.first * r.first + q.first * r.first
    rhs.first = pair_mul(p, r).first + pair_mul(q, r).first
    pair_mul(p, r).first = p.first * r.first
    pair_mul(q, r).first = q.first * r.first
    rhs.first = p.first * r.first + q.first * r.first
    lhs.first = rhs.first
    lhs.second = pair_add(p, q).second * r.second
    pair_add(p, q).second = p.second + q.second
    lhs.second = (p.second + q.second) * r.second
    lhs.second = p.second * r.second + q.second * r.second
    rhs.second = pair_mul(p, r).second + pair_mul(q, r).second
    pair_mul(p, r).second = p.second * r.second
    pair_mul(q, r).second = q.second * r.second
    rhs.second = p.second * r.second + q.second * r.second
    lhs.second = rhs.second
    pair_ext(lhs, rhs)
}

/// Multiplying by componentwise zero on the right gives componentwise zero.
theorem pair_mul_zero_right[A: Semiring, B: Semiring](p: Pair[A, B]) {
    pair_mul(p, pair_zero[A, B]) = pair_zero[A, B]
} by {
    let lhs = pair_mul(p, pair_zero[A, B])
    lhs.first = p.first * pair_zero[A, B].first
    pair_zero[A, B].first = A.0
    lhs.first = p.first * A.0
    lhs.first = A.0
    lhs.second = p.second * pair_zero[A, B].second
    pair_zero[A, B].second = B.0
    lhs.second = p.second * B.0
    lhs.second = B.0
    pair_ext(lhs, pair_zero[A, B])
}

/// Multiplying by componentwise zero on the left gives componentwise zero.
theorem pair_mul_zero_left[A: Semiring, B: Semiring](p: Pair[A, B]) {
    pair_mul(pair_zero[A, B], p) = pair_zero[A, B]
} by {
    let lhs = pair_mul(pair_zero[A, B], p)
    lhs.first = pair_zero[A, B].first * p.first
    pair_zero[A, B].first = A.0
    lhs.first = A.0 * p.first
    lhs.first = A.0
    lhs.second = pair_zero[A, B].second * p.second
    pair_zero[A, B].second = B.0
    lhs.second = B.0 * p.second
    lhs.second = B.0
    pair_ext(lhs, pair_zero[A, B])
}

/// Componentwise inversion is involutive.
theorem pair_inverse_inverse[A: Group, B: Group](p: Pair[A, B]) {
    pair_inverse[A, B](pair_inverse[A, B](p)) = p
} by {
    let lhs = pair_inverse[A, B](pair_inverse[A, B](p))
    lhs.first = pair_inverse[A, B](p).first.inverse
    pair_inverse[A, B](p).first = p.first.inverse
    lhs.first = p.first.inverse.inverse
    inverse_inverse(p.first)
    lhs.first = p.first
    lhs.second = pair_inverse[A, B](p).second.inverse
    pair_inverse[A, B](p).second = p.second.inverse
    lhs.second = p.second.inverse.inverse
    inverse_inverse(p.second)
    lhs.second = p.second
    pair_ext(lhs, p)
}

/// Componentwise inversion reverses componentwise multiplication.
theorem pair_inverse_mul[A: Group, B: Group](p: Pair[A, B], q: Pair[A, B]) {
    pair_inverse[A, B](pair_mul(p, q)) = pair_mul(pair_inverse[A, B](q), pair_inverse[A, B](p))
} by {
    let lhs = pair_inverse[A, B](pair_mul(p, q))
    let rhs = pair_mul(pair_inverse[A, B](q), pair_inverse[A, B](p))
    lhs.first = pair_mul(p, q).first.inverse
    pair_mul(p, q).first = p.first * q.first
    lhs.first = (p.first * q.first).inverse
    inverse_mul(p.first, q.first)
    lhs.first = q.first.inverse * p.first.inverse
    rhs.first = pair_inverse[A, B](q).first * pair_inverse[A, B](p).first
    pair_inverse[A, B](q).first = q.first.inverse
    pair_inverse[A, B](p).first = p.first.inverse
    rhs.first = q.first.inverse * p.first.inverse
    lhs.first = rhs.first
    lhs.second = pair_mul(p, q).second.inverse
    pair_mul(p, q).second = p.second * q.second
    lhs.second = (p.second * q.second).inverse
    inverse_mul(p.second, q.second)
    lhs.second = q.second.inverse * p.second.inverse
    rhs.second = pair_inverse[A, B](q).second * pair_inverse[A, B](p).second
    pair_inverse[A, B](q).second = q.second.inverse
    pair_inverse[A, B](p).second = p.second.inverse
    rhs.second = q.second.inverse * p.second.inverse
    lhs.second = rhs.second
    pair_ext(lhs, rhs)
}

/// Swapping a pair transports componentwise addition.
theorem pair_swap_add[A: Add, B: Add](p: Pair[A, B], q: Pair[A, B]) {
    pair_add(p, q).swap = pair_add(p.swap, q.swap)
} by {
    let lhs = pair_add(p, q).swap
    let rhs = pair_add(p.swap, q.swap)
    lhs.first = pair_add(p, q).second
    lhs.first = p.second + q.second
    rhs.first = p.swap.first + q.swap.first
    rhs.first = p.second + q.second
    lhs.first = rhs.first
    lhs.second = pair_add(p, q).first
    lhs.second = p.first + q.first
    rhs.second = p.swap.second + q.swap.second
    rhs.second = p.first + q.first
    lhs.second = rhs.second
    pair_ext(lhs, rhs)
}

/// Swapping a pair transports componentwise multiplication.
theorem pair_swap_mul[A: Mul, B: Mul](p: Pair[A, B], q: Pair[A, B]) {
    pair_mul(p, q).swap = pair_mul(p.swap, q.swap)
} by {
    let lhs = pair_mul(p, q).swap
    let rhs = pair_mul(p.swap, q.swap)
    lhs.first = pair_mul(p, q).second
    lhs.first = p.second * q.second
    rhs.first = p.swap.first * q.swap.first
    rhs.first = p.second * q.second
    lhs.first = rhs.first
    lhs.second = pair_mul(p, q).first
    lhs.second = p.first * q.first
    rhs.second = p.swap.second * q.swap.second
    rhs.second = p.first * q.first
    lhs.second = rhs.second
    pair_ext(lhs, rhs)
}

/// Swapping a pair transports componentwise zero.
theorem pair_swap_zero[A: Zero, B: Zero] {
    pair_zero[A, B].swap = pair_zero[B, A]
} by {
    let lhs = pair_zero[A, B].swap
    lhs.first = pair_zero[A, B].second
    lhs.first = B.0
    lhs.second = pair_zero[A, B].first
    lhs.second = A.0
    pair_ext(lhs, pair_zero[B, A])
}

/// Swapping a pair transports componentwise one.
theorem pair_swap_one[A: One, B: One] {
    pair_one[A, B].swap = pair_one[B, A]
} by {
    let lhs = pair_one[A, B].swap
    lhs.first = pair_one[A, B].second
    lhs.first = B.1
    lhs.second = pair_one[A, B].first
    lhs.second = A.1
    pair_ext(lhs, pair_one[B, A])
}

/// Swapping a pair transports componentwise negation.
theorem pair_swap_neg[A: Neg, B: Neg](p: Pair[A, B]) {
    pair_neg(p).swap = pair_neg(p.swap)
} by {
    let lhs = pair_neg(p).swap
    let rhs = pair_neg(p.swap)
    lhs.first = pair_neg(p).second
    lhs.first = -p.second
    rhs.first = -p.swap.first
    rhs.first = -p.second
    lhs.first = rhs.first
    lhs.second = pair_neg(p).first
    lhs.second = -p.first
    rhs.second = -p.swap.second
    rhs.second = -p.first
    lhs.second = rhs.second
    pair_ext(lhs, rhs)
}

/// Swapping a pair transports componentwise inversion.
theorem pair_swap_inverse[A: Group, B: Group](p: Pair[A, B]) {
    pair_inverse[A, B](p).swap = pair_inverse[B, A](p.swap)
} by {
    let lhs = pair_inverse[A, B](p).swap
    let rhs = pair_inverse[B, A](p.swap)
    lhs.first = pair_inverse[A, B](p).second
    lhs.first = p.second.inverse
    rhs.first = p.swap.first.inverse
    rhs.first = p.second.inverse
    lhs.first = rhs.first
    lhs.second = pair_inverse[A, B](p).first
    lhs.second = p.first.inverse
    rhs.second = p.swap.second.inverse
    rhs.second = p.first.inverse
    lhs.second = rhs.second
    pair_ext(lhs, rhs)
}
