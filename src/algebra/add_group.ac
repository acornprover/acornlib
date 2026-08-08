from algebra.add_monoid import AddMonoid, AddMonoidHom, is_add_monoid_hom,
    identity_fn_is_add_monoid_hom, compose_is_add_monoid_hom
from algebra.neg import Neg
from data.basic.functions import identity_fn, compose, compose_assoc, compose_identity_left, compose_identity_right,
    function_eq_transport_predicate, function_eq_transport_predicate_rev

/// An additive group is an additive monoid that also has inverses.
typeclass A: AddGroup extends AddMonoid, Neg {
    /// This is what "additive inverse" means.
    inverse_right(a: A) {
        a + -a = A.0
    }
}

attributes A: AddGroup {
    /// Subtracts one element from another using additive inverse.
    define sub(self, other: A) -> A {
        self + -other
    }
}

// This direction is proven rather than assumed
theorem inverse_left[A: AddGroup](a: A) {
    -a + a = A.0
} by {
    (-a + a) + (-a + --a) = -a + --a
}

theorem inverse_inverse[A: AddGroup](a: A) {
    --a = a
} by {
    a + -a + --a = --a
}

theorem left_cancel[A: AddGroup](a: A, b: A, c: A) {
    a + b = a + c implies b = c
} by {
    -a + (a + b) = -a + a + b
    -a + (a + c) = -a + a + c
}

theorem right_cancel[A: AddGroup](a: A, b: A, c: A) {
    b + a = c + a implies b = c
} by {
    b + a + -a = b + (a + -a)
    c + a + -a = c + (a + -a)
}

theorem inverse_add[A: AddGroup](a: A, b: A) {
    -(a + b) = -b + -a
} by {
    -(a + b) + (a + b) = A.0
    -(a + b) + a = -b
}

/// True if a function preserves the additive group operation (is a homomorphism).
define is_add_group_hom[A: AddGroup, B: AddGroup](f: A -> B) -> Bool {
    forall(a: A, b: A) {
        f(a + b) = f(a) + f(b)
    }
}

/// The trivial additive group homomorphism maps every element of A to the identity element of B.
let trivial_add_group_hom[A: AddGroup, B: AddGroup]: A -> B = function(a: A) {
    B.0
}

/// The trivial additive group homomorphism preserves the additive group operation.
theorem trivial_add_group_hom_is_hom[A: AddGroup, B: AddGroup] {
    is_add_group_hom(trivial_add_group_hom[A, B])
}

/// An additive group homomorphism that preserves the additive group structure.
structure AddGroupHom[A: AddGroup, B: AddGroup] {
    /// The mapping for the homomorphism.
    hom: A -> B
} constraint {
    is_add_group_hom(hom)
}

/// Additive group homomorphism extensionality: two homomorphisms are equal when they agree on every input.
theorem add_group_hom_ext[A: AddGroup, B: AddGroup](f: AddGroupHom[A, B], g: AddGroupHom[A, B]) {
    (forall(a: A) { f.hom(a) = g.hom(a) }) implies f = g
} by {
    if forall(a: A) { f.hom(a) = g.hom(a) } {
        f.hom = g.hom
    }
}

attributes AddGroupHom[A: AddGroup, B: AddGroup] {
    /// Additive group homomorphism extensionality from pointwise equality of the homomorphism.
    let ext = add_group_hom_ext[A, B]
}

/// Equal additive group homomorphisms have equal underlying functions.
theorem add_group_hom_eq_hom[A: AddGroup, B: AddGroup](f: AddGroupHom[A, B], g: AddGroupHom[A, B]) {
    f = g implies f.hom = g.hom
}

/// Equal additive group homomorphisms have equal values at every element.
theorem add_group_hom_eq_apply[A: AddGroup, B: AddGroup](f: AddGroupHom[A, B], g: AddGroupHom[A, B], a: A) {
    f = g implies f.hom(a) = g.hom(a)
} by {
    if f = g {
        f.hom(a) = g.hom(a)
    }
}

/// Equality of additive group homomorphisms transports predicates on additive group homomorphisms.
theorem add_group_hom_eq_transport_predicate[A: AddGroup, B: AddGroup](
    p: AddGroupHom[A, B] -> Bool,
    f: AddGroupHom[A, B],
    g: AddGroupHom[A, B]
) {
    f = g and p(f) implies p(g)
}

/// Equality of additive group homomorphisms transports predicates on additive group homomorphisms in the reverse direction.
theorem add_group_hom_eq_transport_predicate_rev[A: AddGroup, B: AddGroup](
    p: AddGroupHom[A, B] -> Bool,
    f: AddGroupHom[A, B],
    g: AddGroupHom[A, B]
) {
    f = g and p(g) implies p(f)
}

/// Equality of underlying functions determines equality of additive group homomorphisms.
theorem add_group_hom_eq_of_hom_eq[A: AddGroup, B: AddGroup](f: AddGroupHom[A, B], g: AddGroupHom[A, B]) {
    f.hom = g.hom implies f = g
} by {
    if f.hom = g.hom {
        forall(a: A) {
            f.hom(a) = g.hom(a)
        }
        add_group_hom_ext(f, g)
    }
}

/// Pointwise equality determines equality of additive group homomorphisms.
theorem add_group_hom_eq_of_apply_eq[A: AddGroup, B: AddGroup](f: AddGroupHom[A, B], g: AddGroupHom[A, B]) {
    (forall(a: A) { f.hom(a) = g.hom(a) }) implies f = g
} by {
    if forall(a: A) { f.hom(a) = g.hom(a) } {
        add_group_hom_ext(f, g)
    }
}

/// An additive group homomorphism preserves the operation.
theorem add_group_hom_add[A: AddGroup, B: AddGroup](f: AddGroupHom[A, B], a: A, b: A) {
    f.hom(a + b) = f.hom(a) + f.hom(b)
} by {
    is_add_group_hom(f.hom)
    is_add_group_hom(f.hom) = forall(x: A, y: A) {
        f.hom(x + y) = f.hom(x) + f.hom(y)
    }
}

/// An additive group homomorphism preserves the identity element.
theorem add_group_hom_zero[A: AddGroup, B: AddGroup](f: AddGroupHom[A, B]) {
    f.hom(A.0) = B.0
} by {
    add_group_hom_add(f, A.0, A.0)
    left_cancel(f.hom(A.0), f.hom(A.0), B.0)
}

/// An additive group homomorphism preserves additive inverses.
theorem add_group_hom_neg[A: AddGroup, B: AddGroup](f: AddGroupHom[A, B], a: A) {
    f.hom(-a) = -f.hom(a)
} by {
    add_group_hom_add(f, a, -a)
    add_group_hom_zero(f)
    left_cancel(f.hom(a), f.hom(-a), -f.hom(a))
}

/// An additive monoid homomorphism between additive groups is an additive group homomorphism.
theorem add_monoid_hom_imp_add_group_hom[A: AddGroup, B: AddGroup](f: A -> B) {
    is_add_monoid_hom(f) implies is_add_group_hom(f)
} by {
    if is_add_monoid_hom(f) {
        is_add_monoid_hom(f) = (f(A.0) = B.0 and forall(a: A, b: A) {
            f(a + b) = f(a) + f(b)
        })
        forall(a: A, b: A) {
            f(a + b) = f(a) + f(b)
        }
        is_add_group_hom(f)
    }
}

/// An additive group homomorphism preserves the full additive monoid structure.
theorem add_group_hom_imp_add_monoid_hom[A: AddGroup, B: AddGroup](f: A -> B) {
    is_add_group_hom(f) implies is_add_monoid_hom(f)
} by {
    if is_add_group_hom(f) {
        is_add_group_hom(f) = forall(x: A, y: A) {
            f(x + y) = f(x) + f(y)
        }
        left_cancel(f(A.0), f(A.0), B.0)
        f(A.0) = B.0
        forall(a: A, b: A) {
            f(a + b) = f(a) + f(b)
        }
        is_add_monoid_hom(f)
    }
}

/// A bundled additive group homomorphism is an additive monoid homomorphism.
theorem add_group_hom_is_add_monoid_hom[A: AddGroup, B: AddGroup](f: AddGroupHom[A, B]) {
    is_add_monoid_hom(f.hom)
} by {
    add_group_hom_imp_add_monoid_hom(f.hom)
}

/// A bundled additive monoid homomorphism between additive groups is an additive group homomorphism.
theorem add_monoid_hom_is_add_group_hom[A: AddGroup, B: AddGroup](f: AddMonoidHom[A, B]) {
    is_add_group_hom(f.hom)
} by {
    add_monoid_hom_imp_add_group_hom(f.hom)
}

/// The additive monoid homomorphism underlying an additive group homomorphism.
let add_group_hom_to_add_monoid_hom[A: AddGroup, B: AddGroup](f: AddGroupHom[A, B]) -> result: AddMonoidHom[A, B] satisfy {
    AddMonoidHom.new(f.hom) = Option.some(result)
} by {
    add_group_hom_is_add_monoid_hom(f)
}

/// The underlying function of the additive monoid homomorphism associated to an additive group homomorphism.
theorem add_group_hom_to_add_monoid_hom_hom[A: AddGroup, B: AddGroup](f: AddGroupHom[A, B]) {
    add_group_hom_to_add_monoid_hom(f).hom = f.hom
}

/// The additive group homomorphism associated to an additive monoid homomorphism between additive groups.
let add_group_hom_of_add_monoid_hom[A: AddGroup, B: AddGroup](f: AddMonoidHom[A, B]) -> result: AddGroupHom[A, B] satisfy {
    AddGroupHom.new(f.hom) = Option.some(result)
} by {
    add_monoid_hom_is_add_group_hom(f)
}

/// The underlying function of the additive group homomorphism associated to an additive monoid homomorphism.
theorem add_group_hom_of_add_monoid_hom_hom[A: AddGroup, B: AddGroup](f: AddMonoidHom[A, B]) {
    add_group_hom_of_add_monoid_hom(f).hom = f.hom
}

/// Equality of functions transports the property of being an additive group homomorphism.
theorem function_eq_transport_add_group_hom[A: AddGroup, B: AddGroup](f: A -> B, g: A -> B) {
    f = g and is_add_group_hom(f) implies is_add_group_hom(g)
} by {
    if f = g and is_add_group_hom(f) {
        function_eq_transport_predicate(is_add_group_hom[A, B], f, g)
    }
}

/// Equality of functions transports the property of being an additive group homomorphism in the reverse direction.
theorem function_eq_transport_add_group_hom_rev[A: AddGroup, B: AddGroup](f: A -> B, g: A -> B) {
    f = g and is_add_group_hom(g) implies is_add_group_hom(f)
} by {
    if f = g and is_add_group_hom(g) {
        function_eq_transport_predicate_rev(is_add_group_hom[A, B], f, g)
    }
}

/// The identity function on an additive group preserves the additive group operation.
theorem identity_fn_is_add_group_hom[A: AddGroup] {
    is_add_group_hom(identity_fn[A])
} by {
    identity_fn_is_add_monoid_hom[A]
    add_monoid_hom_imp_add_group_hom(identity_fn[A])
}

/// The composition of two additive group homomorphisms preserves the additive group operation.
theorem compose_is_add_group_hom[A: AddGroup, B: AddGroup, C: AddGroup](f: B -> C, g: A -> B) {
    is_add_group_hom(f) and is_add_group_hom(g) implies is_add_group_hom(compose(f, g))
} by {
    if is_add_group_hom(f) and is_add_group_hom(g) {
        add_group_hom_imp_add_monoid_hom(f)
        add_group_hom_imp_add_monoid_hom(g)
        compose_is_add_monoid_hom(f, g)
        add_monoid_hom_imp_add_group_hom(compose(f, g))
        is_add_group_hom(compose(f, g))
    }
}

/// The identity homomorphism on an additive group, which sends every element to itself.
let identity_add_group_hom[A: AddGroup]: AddGroupHom[A, A] satisfy {
    AddGroupHom.new(identity_fn[A]) = Option.some(identity_add_group_hom)
}

/// The underlying function of the identity additive group homomorphism is the identity function.
theorem identity_add_group_hom_hom[A: AddGroup] {
    identity_add_group_hom[A].hom = identity_fn[A]
}

/// The composition of two additive group homomorphisms.
let compose_add_group_hom[A: AddGroup, B: AddGroup, C: AddGroup](f: AddGroupHom[B, C], g: AddGroupHom[A, B]) -> result: AddGroupHom[A, C] satisfy {
    AddGroupHom.new(compose(f.hom, g.hom)) = Option.some(result)
} by {
    compose_is_add_group_hom(f.hom, g.hom)
}

/// The underlying function of a composition of additive group homomorphisms is the composition of underlying functions.
theorem compose_add_group_hom_hom[A: AddGroup, B: AddGroup, C: AddGroup](f: AddGroupHom[B, C], g: AddGroupHom[A, B]) {
    compose_add_group_hom(f, g).hom = compose(f.hom, g.hom)
}

/// Composition of additive group homomorphisms is associative.
theorem compose_add_group_hom_assoc[A: AddGroup, B: AddGroup, C: AddGroup, D: AddGroup](
    f: AddGroupHom[C, D], g: AddGroupHom[B, C], h: AddGroupHom[A, B]) {
    compose_add_group_hom(compose_add_group_hom(f, g), h) = compose_add_group_hom(f, compose_add_group_hom(g, h))
} by {
    let lhs = compose_add_group_hom(compose_add_group_hom(f, g), h)
    let rhs = compose_add_group_hom(f, compose_add_group_hom(g, h))
    lhs.hom = compose(compose_add_group_hom(f, g).hom, h.hom)
    compose_add_group_hom(f, g).hom = compose(f.hom, g.hom)
    lhs.hom = compose(compose(f.hom, g.hom), h.hom)
    rhs.hom = compose(f.hom, compose_add_group_hom(g, h).hom)
    compose_add_group_hom(g, h).hom = compose(g.hom, h.hom)
    rhs.hom = compose(f.hom, compose(g.hom, h.hom))
    compose_assoc(f.hom, g.hom, h.hom)
    lhs.hom = rhs.hom
}

/// Composing the identity additive group homomorphism on the left does nothing.
theorem compose_add_group_hom_identity_left[A: AddGroup, B: AddGroup](f: AddGroupHom[A, B]) {
    compose_add_group_hom(identity_add_group_hom[B], f) = f
} by {
    let lhs = compose_add_group_hom(identity_add_group_hom[B], f)
    compose_identity_left(f.hom)
    lhs.hom = f.hom
}

/// Composing the identity additive group homomorphism on the right does nothing.
theorem compose_add_group_hom_identity_right[A: AddGroup, B: AddGroup](f: AddGroupHom[A, B]) {
    compose_add_group_hom(f, identity_add_group_hom[A]) = f
} by {
    let lhs = compose_add_group_hom(f, identity_add_group_hom[A])
    compose_identity_right(f.hom)
    lhs.hom = f.hom
}
