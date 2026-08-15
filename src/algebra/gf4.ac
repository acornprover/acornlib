/// The field GF(4) = F_2[x]/(x^2 + x + 1) with four elements.
/// The four elements 0, 1, X, X+1 carry the quotient operations modulo the
/// irreducible polynomial x^2 + x + 1 over F_2, so X^2 + X + 1 = 0.
///

from nat import Nat
from algebra.add import Add
from algebra.mul import Mul
from algebra.neg import Neg
from algebra.zero import Zero
from algebra.one import One
from algebra.inverse import Inverse
from algebra.add_semigroup import AddSemigroup
from algebra.add_comm_semigroup import AddCommSemigroup
from algebra.add_monoid import AddMonoid
from algebra.add_comm_monoid import AddCommMonoid
from algebra.add_group import AddGroup
from algebra.add_comm_group import AddCommGroup
from algebra.semigroup import Semigroup
from algebra.comm_semigroup import CommSemigroup
from algebra.monoid.monoid import Monoid
from algebra.comm_monoid import CommMonoid
from semiring import Semiring
from algebra.ring.ring import Ring
from comm_ring import CommRing
from algebra.field.field import Field

/// The finite field GF4 = F_2[x]/(irreducible polynomial x^2 + x + 1 over F_2, so X^2 + X + 1 = 0.), as an enumerated type with four
/// elements; the operations below are the quotient operations modulo
/// the modulus.
inductive GF4 {
    /// The class of zero.
    zero
    /// The class of one.
    one
    /// The class of X.
    x
    /// The class of X + 1.
    x_plus_one
}

/// Addition in GF4: coefficient-wise addition in F_2.
define gf4_add(x: GF4, y: GF4) -> GF4 {
    match x {
        GF4.zero {
            match y {
                GF4.zero { GF4.zero }
                GF4.one { GF4.one }
                GF4.x { GF4.x }
                GF4.x_plus_one { GF4.x_plus_one }
            }
        }
        GF4.one {
            match y {
                GF4.zero { GF4.one }
                GF4.one { GF4.zero }
                GF4.x { GF4.x_plus_one }
                GF4.x_plus_one { GF4.x }
            }
        }
        GF4.x {
            match y {
                GF4.zero { GF4.x }
                GF4.one { GF4.x_plus_one }
                GF4.x { GF4.zero }
                GF4.x_plus_one { GF4.one }
            }
        }
        GF4.x_plus_one {
            match y {
                GF4.zero { GF4.x_plus_one }
                GF4.one { GF4.x }
                GF4.x { GF4.one }
                GF4.x_plus_one { GF4.zero }
            }
        }
    }
}

/// Multiplication in GF4 modulo the defining polynomial.
define gf4_mul(x: GF4, y: GF4) -> GF4 {
    match x {
        GF4.zero {
            match y {
                GF4.zero { GF4.zero }
                GF4.one { GF4.zero }
                GF4.x { GF4.zero }
                GF4.x_plus_one { GF4.zero }
            }
        }
        GF4.one {
            match y {
                GF4.zero { GF4.zero }
                GF4.one { GF4.one }
                GF4.x { GF4.x }
                GF4.x_plus_one { GF4.x_plus_one }
            }
        }
        GF4.x {
            match y {
                GF4.zero { GF4.zero }
                GF4.one { GF4.x }
                GF4.x { GF4.x_plus_one }
                GF4.x_plus_one { GF4.one }
            }
        }
        GF4.x_plus_one {
            match y {
                GF4.zero { GF4.zero }
                GF4.one { GF4.x_plus_one }
                GF4.x { GF4.one }
                GF4.x_plus_one { GF4.x }
            }
        }
    }
}

/// Additive inverse in GF4 (the identity in characteristic two).
define gf4_neg(x: GF4) -> GF4 {
    match x {
        GF4.zero { GF4.zero }
        GF4.one { GF4.one }
        GF4.x { GF4.x }
        GF4.x_plus_one { GF4.x_plus_one }
    }
}

/// Multiplicative inverse in GF4; the inverse of zero is zero.
define gf4_inv(x: GF4) -> GF4 {
    match x {
        GF4.zero { GF4.zero }
        GF4.one { GF4.one }
        GF4.x { GF4.x_plus_one }
        GF4.x_plus_one { GF4.x }
    }
}

/// GF4 elements add.
instance GF4: Add {
    let add = gf4_add
}

/// GF4 elements multiply.
instance GF4: Mul {
    let mul = gf4_mul
}

/// GF4 elements have additive inverses.
instance GF4: Neg {
    let neg = gf4_neg
}

/// The zero of GF4.
instance GF4: Zero {
    let 0 = GF4.zero
}

/// The one of GF4.
instance GF4: One {
    let 1 = GF4.one
}

/// Multiplicative inverses in GF4.
instance GF4: Inverse {
    define inverse(self) -> GF4 {
        gf4_inv(self)
    }
}

/// Every element of GF4 is one of its 4 constructors.
theorem gf4_cases(x: GF4) {
    x = GF4.zero or x = GF4.one or x = GF4.x or x = GF4.x_plus_one
} by {
    GF4.induction(function(y: GF4) {
        y = GF4.zero or y = GF4.one or y = GF4.x or y = GF4.x_plus_one
    })
}

/// gf4_add_zero_left: the identity/zero law at the level of elements.
theorem gf4_add_zero_left(z: GF4) {
    Add.add(Zero.0[GF4], z) = z
} by {
    define w(a: GF4) -> Bool {
        gf4_add(GF4.zero, a) = a
    }
    gf4_add(GF4.zero, GF4.zero) = GF4.zero
    gf4_add(GF4.zero, GF4.one) = GF4.one
    gf4_add(GF4.zero, GF4.x) = GF4.x
    gf4_add(GF4.zero, GF4.x_plus_one) = GF4.x_plus_one
    w(GF4.zero)
    w(GF4.one)
    w(GF4.x)
    w(GF4.x_plus_one)
    GF4.induction(w)
    w(z)
    gf4_add(GF4.zero, z) = z
    Add.add(Zero.0[GF4], z) = z
}

/// gf4_add_zero_right: the identity/zero law at the level of elements.
theorem gf4_add_zero_right(z: GF4) {
    Add.add(z, Zero.0[GF4]) = z
} by {
    define w(a: GF4) -> Bool {
        gf4_add(a, GF4.zero) = a
    }
    gf4_add(GF4.zero, GF4.zero) = GF4.zero
    gf4_add(GF4.one, GF4.zero) = GF4.one
    gf4_add(GF4.x, GF4.zero) = GF4.x
    gf4_add(GF4.x_plus_one, GF4.zero) = GF4.x_plus_one
    w(GF4.zero)
    w(GF4.one)
    w(GF4.x)
    w(GF4.x_plus_one)
    GF4.induction(w)
    w(z)
    gf4_add(z, GF4.zero) = z
    Add.add(z, Zero.0[GF4]) = z
}

/// gf4_add_neg_right: the identity/zero law at the level of elements.
theorem gf4_add_neg_right(z: GF4) {
    Add.add(z, Neg.neg(z)) = Zero.0[GF4]
} by {
    define w(a: GF4) -> Bool {
        gf4_add(a, gf4_neg(a)) = GF4.zero
    }
    gf4_add(GF4.zero, gf4_neg(GF4.zero)) = GF4.zero
    gf4_add(GF4.one, gf4_neg(GF4.one)) = GF4.zero
    gf4_add(GF4.x, gf4_neg(GF4.x)) = GF4.zero
    gf4_add(GF4.x_plus_one, gf4_neg(GF4.x_plus_one)) = GF4.zero
    w(GF4.zero)
    w(GF4.one)
    w(GF4.x)
    w(GF4.x_plus_one)
    GF4.induction(w)
    w(z)
    gf4_add(z, gf4_neg(z)) = GF4.zero
    Add.add(z, Neg.neg(z)) = Zero.0[GF4]
}

/// gf4_mul_one_left: the identity/zero law at the level of elements.
theorem gf4_mul_one_left(z: GF4) {
    Mul.mul(One.1[GF4], z) = z
} by {
    define w(a: GF4) -> Bool {
        gf4_mul(GF4.one, a) = a
    }
    gf4_mul(GF4.one, GF4.zero) = GF4.zero
    gf4_mul(GF4.one, GF4.one) = GF4.one
    gf4_mul(GF4.one, GF4.x) = GF4.x
    gf4_mul(GF4.one, GF4.x_plus_one) = GF4.x_plus_one
    w(GF4.zero)
    w(GF4.one)
    w(GF4.x)
    w(GF4.x_plus_one)
    GF4.induction(w)
    w(z)
    gf4_mul(GF4.one, z) = z
    Mul.mul(One.1[GF4], z) = z
}

/// gf4_mul_one_right: the identity/zero law at the level of elements.
theorem gf4_mul_one_right(z: GF4) {
    Mul.mul(z, One.1[GF4]) = z
} by {
    define w(a: GF4) -> Bool {
        gf4_mul(a, GF4.one) = a
    }
    gf4_mul(GF4.zero, GF4.one) = GF4.zero
    gf4_mul(GF4.one, GF4.one) = GF4.one
    gf4_mul(GF4.x, GF4.one) = GF4.x
    gf4_mul(GF4.x_plus_one, GF4.one) = GF4.x_plus_one
    w(GF4.zero)
    w(GF4.one)
    w(GF4.x)
    w(GF4.x_plus_one)
    GF4.induction(w)
    w(z)
    gf4_mul(z, GF4.one) = z
    Mul.mul(z, One.1[GF4]) = z
}

/// gf4_mul_zero_left: the identity/zero law at the level of elements.
theorem gf4_mul_zero_left(z: GF4) {
    Mul.mul(Zero.0[GF4], z) = Zero.0[GF4]
} by {
    define w(a: GF4) -> Bool {
        gf4_mul(GF4.zero, a) = GF4.zero
    }
    gf4_mul(GF4.zero, GF4.zero) = GF4.zero
    gf4_mul(GF4.zero, GF4.one) = GF4.zero
    gf4_mul(GF4.zero, GF4.x) = GF4.zero
    gf4_mul(GF4.zero, GF4.x_plus_one) = GF4.zero
    w(GF4.zero)
    w(GF4.one)
    w(GF4.x)
    w(GF4.x_plus_one)
    GF4.induction(w)
    w(z)
    gf4_mul(GF4.zero, z) = GF4.zero
    Mul.mul(Zero.0[GF4], z) = Zero.0[GF4]
}

/// gf4_mul_zero_right: the identity/zero law at the level of elements.
theorem gf4_mul_zero_right(z: GF4) {
    Mul.mul(z, Zero.0[GF4]) = Zero.0[GF4]
} by {
    define w(a: GF4) -> Bool {
        gf4_mul(a, GF4.zero) = GF4.zero
    }
    gf4_mul(GF4.zero, GF4.zero) = GF4.zero
    gf4_mul(GF4.one, GF4.zero) = GF4.zero
    gf4_mul(GF4.x, GF4.zero) = GF4.zero
    gf4_mul(GF4.x_plus_one, GF4.zero) = GF4.zero
    w(GF4.zero)
    w(GF4.one)
    w(GF4.x)
    w(GF4.x_plus_one)
    GF4.induction(w)
    w(z)
    gf4_mul(z, GF4.zero) = GF4.zero
    Mul.mul(z, Zero.0[GF4]) = Zero.0[GF4]
}

/// gf4_add_comm with the first argument fixed at GF4.zero.
theorem gf4_add_comm_e0(z: GF4) {
    gf4_add(GF4.zero, z) = gf4_add(z, GF4.zero)
} by {
    define w(a: GF4) -> Bool {
        gf4_add(GF4.zero, a) = gf4_add(a, GF4.zero)
    }
    gf4_add(GF4.zero, GF4.zero) = gf4_add(GF4.zero, GF4.zero)
    gf4_add(GF4.zero, GF4.one) = gf4_add(GF4.one, GF4.zero)
    gf4_add(GF4.zero, GF4.x) = gf4_add(GF4.x, GF4.zero)
    gf4_add(GF4.zero, GF4.x_plus_one) = gf4_add(GF4.x_plus_one, GF4.zero)
    w(GF4.zero)
    w(GF4.one)
    w(GF4.x)
    w(GF4.x_plus_one)
    GF4.induction(w)
    w(z)
    gf4_add(GF4.zero, z) = gf4_add(z, GF4.zero)
}

/// gf4_add_comm with the first argument fixed at GF4.one.
theorem gf4_add_comm_e1(z: GF4) {
    gf4_add(GF4.one, z) = gf4_add(z, GF4.one)
} by {
    define w(a: GF4) -> Bool {
        gf4_add(GF4.one, a) = gf4_add(a, GF4.one)
    }
    gf4_add(GF4.one, GF4.zero) = gf4_add(GF4.zero, GF4.one)
    gf4_add(GF4.one, GF4.one) = gf4_add(GF4.one, GF4.one)
    gf4_add(GF4.one, GF4.x) = gf4_add(GF4.x, GF4.one)
    gf4_add(GF4.one, GF4.x_plus_one) = gf4_add(GF4.x_plus_one, GF4.one)
    w(GF4.zero)
    w(GF4.one)
    w(GF4.x)
    w(GF4.x_plus_one)
    GF4.induction(w)
    w(z)
    gf4_add(GF4.one, z) = gf4_add(z, GF4.one)
}

/// gf4_add_comm with the first argument fixed at GF4.x.
theorem gf4_add_comm_e2(z: GF4) {
    gf4_add(GF4.x, z) = gf4_add(z, GF4.x)
} by {
    define w(a: GF4) -> Bool {
        gf4_add(GF4.x, a) = gf4_add(a, GF4.x)
    }
    gf4_add(GF4.x, GF4.zero) = gf4_add(GF4.zero, GF4.x)
    gf4_add(GF4.x, GF4.one) = gf4_add(GF4.one, GF4.x)
    gf4_add(GF4.x, GF4.x) = gf4_add(GF4.x, GF4.x)
    gf4_add(GF4.x, GF4.x_plus_one) = gf4_add(GF4.x_plus_one, GF4.x)
    w(GF4.zero)
    w(GF4.one)
    w(GF4.x)
    w(GF4.x_plus_one)
    GF4.induction(w)
    w(z)
    gf4_add(GF4.x, z) = gf4_add(z, GF4.x)
}

/// gf4_add_comm with the first argument fixed at GF4.x_plus_one.
theorem gf4_add_comm_e3(z: GF4) {
    gf4_add(GF4.x_plus_one, z) = gf4_add(z, GF4.x_plus_one)
} by {
    define w(a: GF4) -> Bool {
        gf4_add(GF4.x_plus_one, a) = gf4_add(a, GF4.x_plus_one)
    }
    gf4_add(GF4.x_plus_one, GF4.zero) = gf4_add(GF4.zero, GF4.x_plus_one)
    gf4_add(GF4.x_plus_one, GF4.one) = gf4_add(GF4.one, GF4.x_plus_one)
    gf4_add(GF4.x_plus_one, GF4.x) = gf4_add(GF4.x, GF4.x_plus_one)
    gf4_add(GF4.x_plus_one, GF4.x_plus_one) = gf4_add(GF4.x_plus_one, GF4.x_plus_one)
    w(GF4.zero)
    w(GF4.one)
    w(GF4.x)
    w(GF4.x_plus_one)
    GF4.induction(w)
    w(z)
    gf4_add(GF4.x_plus_one, z) = gf4_add(z, GF4.x_plus_one)
}

/// gf4_add_comm.
theorem gf4_add_comm(x: GF4, y: GF4) {
    Add.add(x, y) = Add.add(y, x)
} by {
    gf4_cases(x)
    if x = GF4.zero {
        gf4_add_comm_e0(y)
        gf4_add(GF4.zero, y) = gf4_add(y, GF4.zero)
        gf4_add(x, y) = gf4_add(y, x)
    }
    if x = GF4.one {
        gf4_add_comm_e1(y)
        gf4_add(GF4.one, y) = gf4_add(y, GF4.one)
        gf4_add(x, y) = gf4_add(y, x)
    }
    if x = GF4.x {
        gf4_add_comm_e2(y)
        gf4_add(GF4.x, y) = gf4_add(y, GF4.x)
        gf4_add(x, y) = gf4_add(y, x)
    }
    if x = GF4.x_plus_one {
        gf4_add_comm_e3(y)
        gf4_add(GF4.x_plus_one, y) = gf4_add(y, GF4.x_plus_one)
        gf4_add(x, y) = gf4_add(y, x)
    }
    gf4_add(x, y) = gf4_add(y, x)
    Add.add(x, y) = Add.add(y, x)
}

/// gf4_mul_comm with the first argument fixed at GF4.zero.
theorem gf4_mul_comm_e0(z: GF4) {
    gf4_mul(GF4.zero, z) = gf4_mul(z, GF4.zero)
} by {
    define w(a: GF4) -> Bool {
        gf4_mul(GF4.zero, a) = gf4_mul(a, GF4.zero)
    }
    gf4_mul(GF4.zero, GF4.zero) = gf4_mul(GF4.zero, GF4.zero)
    gf4_mul(GF4.zero, GF4.one) = gf4_mul(GF4.one, GF4.zero)
    gf4_mul(GF4.zero, GF4.x) = gf4_mul(GF4.x, GF4.zero)
    gf4_mul(GF4.zero, GF4.x_plus_one) = gf4_mul(GF4.x_plus_one, GF4.zero)
    w(GF4.zero)
    w(GF4.one)
    w(GF4.x)
    w(GF4.x_plus_one)
    GF4.induction(w)
    w(z)
    gf4_mul(GF4.zero, z) = gf4_mul(z, GF4.zero)
}

/// gf4_mul_comm with the first argument fixed at GF4.one.
theorem gf4_mul_comm_e1(z: GF4) {
    gf4_mul(GF4.one, z) = gf4_mul(z, GF4.one)
} by {
    define w(a: GF4) -> Bool {
        gf4_mul(GF4.one, a) = gf4_mul(a, GF4.one)
    }
    gf4_mul(GF4.one, GF4.zero) = gf4_mul(GF4.zero, GF4.one)
    gf4_mul(GF4.one, GF4.one) = gf4_mul(GF4.one, GF4.one)
    gf4_mul(GF4.one, GF4.x) = gf4_mul(GF4.x, GF4.one)
    gf4_mul(GF4.one, GF4.x_plus_one) = gf4_mul(GF4.x_plus_one, GF4.one)
    w(GF4.zero)
    w(GF4.one)
    w(GF4.x)
    w(GF4.x_plus_one)
    GF4.induction(w)
    w(z)
    gf4_mul(GF4.one, z) = gf4_mul(z, GF4.one)
}

/// gf4_mul_comm with the first argument fixed at GF4.x.
theorem gf4_mul_comm_e2(z: GF4) {
    gf4_mul(GF4.x, z) = gf4_mul(z, GF4.x)
} by {
    define w(a: GF4) -> Bool {
        gf4_mul(GF4.x, a) = gf4_mul(a, GF4.x)
    }
    gf4_mul(GF4.x, GF4.zero) = gf4_mul(GF4.zero, GF4.x)
    gf4_mul(GF4.x, GF4.one) = gf4_mul(GF4.one, GF4.x)
    gf4_mul(GF4.x, GF4.x) = gf4_mul(GF4.x, GF4.x)
    gf4_mul(GF4.x, GF4.x_plus_one) = gf4_mul(GF4.x_plus_one, GF4.x)
    w(GF4.zero)
    w(GF4.one)
    w(GF4.x)
    w(GF4.x_plus_one)
    GF4.induction(w)
    w(z)
    gf4_mul(GF4.x, z) = gf4_mul(z, GF4.x)
}

/// gf4_mul_comm with the first argument fixed at GF4.x_plus_one.
theorem gf4_mul_comm_e3(z: GF4) {
    gf4_mul(GF4.x_plus_one, z) = gf4_mul(z, GF4.x_plus_one)
} by {
    define w(a: GF4) -> Bool {
        gf4_mul(GF4.x_plus_one, a) = gf4_mul(a, GF4.x_plus_one)
    }
    gf4_mul(GF4.x_plus_one, GF4.zero) = gf4_mul(GF4.zero, GF4.x_plus_one)
    gf4_mul(GF4.x_plus_one, GF4.one) = gf4_mul(GF4.one, GF4.x_plus_one)
    gf4_mul(GF4.x_plus_one, GF4.x) = gf4_mul(GF4.x, GF4.x_plus_one)
    gf4_mul(GF4.x_plus_one, GF4.x_plus_one) = gf4_mul(GF4.x_plus_one, GF4.x_plus_one)
    w(GF4.zero)
    w(GF4.one)
    w(GF4.x)
    w(GF4.x_plus_one)
    GF4.induction(w)
    w(z)
    gf4_mul(GF4.x_plus_one, z) = gf4_mul(z, GF4.x_plus_one)
}

/// gf4_mul_comm.
theorem gf4_mul_comm(x: GF4, y: GF4) {
    Mul.mul(x, y) = Mul.mul(y, x)
} by {
    gf4_cases(x)
    if x = GF4.zero {
        gf4_mul_comm_e0(y)
        gf4_mul(GF4.zero, y) = gf4_mul(y, GF4.zero)
        gf4_mul(x, y) = gf4_mul(y, x)
    }
    if x = GF4.one {
        gf4_mul_comm_e1(y)
        gf4_mul(GF4.one, y) = gf4_mul(y, GF4.one)
        gf4_mul(x, y) = gf4_mul(y, x)
    }
    if x = GF4.x {
        gf4_mul_comm_e2(y)
        gf4_mul(GF4.x, y) = gf4_mul(y, GF4.x)
        gf4_mul(x, y) = gf4_mul(y, x)
    }
    if x = GF4.x_plus_one {
        gf4_mul_comm_e3(y)
        gf4_mul(GF4.x_plus_one, y) = gf4_mul(y, GF4.x_plus_one)
        gf4_mul(x, y) = gf4_mul(y, x)
    }
    gf4_mul(x, y) = gf4_mul(y, x)
    Mul.mul(x, y) = Mul.mul(y, x)
}

/// gf4_add_assoc with the first two arguments fixed.
theorem gf4_add_assoc_e0e0(z: GF4) {
    gf4_add(GF4.zero, gf4_add(GF4.zero, z)) = gf4_add(gf4_add(GF4.zero, GF4.zero), z)
} by {
    define w(a: GF4) -> Bool {
        gf4_add(GF4.zero, gf4_add(GF4.zero, a)) = gf4_add(gf4_add(GF4.zero, GF4.zero), a)
    }
    gf4_add(GF4.zero, gf4_add(GF4.zero, GF4.zero)) = gf4_add(gf4_add(GF4.zero, GF4.zero), GF4.zero)
    gf4_add(GF4.zero, gf4_add(GF4.zero, GF4.one)) = gf4_add(gf4_add(GF4.zero, GF4.zero), GF4.one)
    gf4_add(GF4.zero, gf4_add(GF4.zero, GF4.x)) = gf4_add(gf4_add(GF4.zero, GF4.zero), GF4.x)
    gf4_add(GF4.zero, gf4_add(GF4.zero, GF4.x_plus_one)) = gf4_add(gf4_add(GF4.zero, GF4.zero), GF4.x_plus_one)
    w(GF4.zero)
    w(GF4.one)
    w(GF4.x)
    w(GF4.x_plus_one)
    GF4.induction(w)
    w(z)
    gf4_add(GF4.zero, gf4_add(GF4.zero, z)) = gf4_add(gf4_add(GF4.zero, GF4.zero), z)
}

/// gf4_add_assoc with the first two arguments fixed.
theorem gf4_add_assoc_e0e1(z: GF4) {
    gf4_add(GF4.zero, gf4_add(GF4.one, z)) = gf4_add(gf4_add(GF4.zero, GF4.one), z)
} by {
    define w(a: GF4) -> Bool {
        gf4_add(GF4.zero, gf4_add(GF4.one, a)) = gf4_add(gf4_add(GF4.zero, GF4.one), a)
    }
    gf4_add(GF4.zero, gf4_add(GF4.one, GF4.zero)) = gf4_add(gf4_add(GF4.zero, GF4.one), GF4.zero)
    gf4_add(GF4.zero, gf4_add(GF4.one, GF4.one)) = gf4_add(gf4_add(GF4.zero, GF4.one), GF4.one)
    gf4_add(GF4.zero, gf4_add(GF4.one, GF4.x)) = gf4_add(gf4_add(GF4.zero, GF4.one), GF4.x)
    gf4_add(GF4.zero, gf4_add(GF4.one, GF4.x_plus_one)) = gf4_add(gf4_add(GF4.zero, GF4.one), GF4.x_plus_one)
    w(GF4.zero)
    w(GF4.one)
    w(GF4.x)
    w(GF4.x_plus_one)
    GF4.induction(w)
    w(z)
    gf4_add(GF4.zero, gf4_add(GF4.one, z)) = gf4_add(gf4_add(GF4.zero, GF4.one), z)
}

/// gf4_add_assoc with the first two arguments fixed.
theorem gf4_add_assoc_e0e2(z: GF4) {
    gf4_add(GF4.zero, gf4_add(GF4.x, z)) = gf4_add(gf4_add(GF4.zero, GF4.x), z)
} by {
    define w(a: GF4) -> Bool {
        gf4_add(GF4.zero, gf4_add(GF4.x, a)) = gf4_add(gf4_add(GF4.zero, GF4.x), a)
    }
    gf4_add(GF4.zero, gf4_add(GF4.x, GF4.zero)) = gf4_add(gf4_add(GF4.zero, GF4.x), GF4.zero)
    gf4_add(GF4.zero, gf4_add(GF4.x, GF4.one)) = gf4_add(gf4_add(GF4.zero, GF4.x), GF4.one)
    gf4_add(GF4.zero, gf4_add(GF4.x, GF4.x)) = gf4_add(gf4_add(GF4.zero, GF4.x), GF4.x)
    gf4_add(GF4.zero, gf4_add(GF4.x, GF4.x_plus_one)) = gf4_add(gf4_add(GF4.zero, GF4.x), GF4.x_plus_one)
    w(GF4.zero)
    w(GF4.one)
    w(GF4.x)
    w(GF4.x_plus_one)
    GF4.induction(w)
    w(z)
    gf4_add(GF4.zero, gf4_add(GF4.x, z)) = gf4_add(gf4_add(GF4.zero, GF4.x), z)
}

/// gf4_add_assoc with the first two arguments fixed.
theorem gf4_add_assoc_e0e3(z: GF4) {
    gf4_add(GF4.zero, gf4_add(GF4.x_plus_one, z)) = gf4_add(gf4_add(GF4.zero, GF4.x_plus_one), z)
} by {
    define w(a: GF4) -> Bool {
        gf4_add(GF4.zero, gf4_add(GF4.x_plus_one, a)) = gf4_add(gf4_add(GF4.zero, GF4.x_plus_one), a)
    }
    gf4_add(GF4.zero, gf4_add(GF4.x_plus_one, GF4.zero)) = gf4_add(gf4_add(GF4.zero, GF4.x_plus_one), GF4.zero)
    gf4_add(GF4.zero, gf4_add(GF4.x_plus_one, GF4.one)) = gf4_add(gf4_add(GF4.zero, GF4.x_plus_one), GF4.one)
    gf4_add(GF4.zero, gf4_add(GF4.x_plus_one, GF4.x)) = gf4_add(gf4_add(GF4.zero, GF4.x_plus_one), GF4.x)
    gf4_add(GF4.zero, gf4_add(GF4.x_plus_one, GF4.x_plus_one)) = gf4_add(gf4_add(GF4.zero, GF4.x_plus_one), GF4.x_plus_one)
    w(GF4.zero)
    w(GF4.one)
    w(GF4.x)
    w(GF4.x_plus_one)
    GF4.induction(w)
    w(z)
    gf4_add(GF4.zero, gf4_add(GF4.x_plus_one, z)) = gf4_add(gf4_add(GF4.zero, GF4.x_plus_one), z)
}

/// gf4_add_assoc with the first two arguments fixed.
theorem gf4_add_assoc_e1e0(z: GF4) {
    gf4_add(GF4.one, gf4_add(GF4.zero, z)) = gf4_add(gf4_add(GF4.one, GF4.zero), z)
} by {
    define w(a: GF4) -> Bool {
        gf4_add(GF4.one, gf4_add(GF4.zero, a)) = gf4_add(gf4_add(GF4.one, GF4.zero), a)
    }
    gf4_add(GF4.one, gf4_add(GF4.zero, GF4.zero)) = gf4_add(gf4_add(GF4.one, GF4.zero), GF4.zero)
    gf4_add(GF4.one, gf4_add(GF4.zero, GF4.one)) = gf4_add(gf4_add(GF4.one, GF4.zero), GF4.one)
    gf4_add(GF4.one, gf4_add(GF4.zero, GF4.x)) = gf4_add(gf4_add(GF4.one, GF4.zero), GF4.x)
    gf4_add(GF4.one, gf4_add(GF4.zero, GF4.x_plus_one)) = gf4_add(gf4_add(GF4.one, GF4.zero), GF4.x_plus_one)
    w(GF4.zero)
    w(GF4.one)
    w(GF4.x)
    w(GF4.x_plus_one)
    GF4.induction(w)
    w(z)
    gf4_add(GF4.one, gf4_add(GF4.zero, z)) = gf4_add(gf4_add(GF4.one, GF4.zero), z)
}

/// gf4_add_assoc with the first two arguments fixed.
theorem gf4_add_assoc_e1e1(z: GF4) {
    gf4_add(GF4.one, gf4_add(GF4.one, z)) = gf4_add(gf4_add(GF4.one, GF4.one), z)
} by {
    define w(a: GF4) -> Bool {
        gf4_add(GF4.one, gf4_add(GF4.one, a)) = gf4_add(gf4_add(GF4.one, GF4.one), a)
    }
    gf4_add(GF4.one, gf4_add(GF4.one, GF4.zero)) = gf4_add(gf4_add(GF4.one, GF4.one), GF4.zero)
    gf4_add(GF4.one, gf4_add(GF4.one, GF4.one)) = gf4_add(gf4_add(GF4.one, GF4.one), GF4.one)
    gf4_add(GF4.one, gf4_add(GF4.one, GF4.x)) = gf4_add(gf4_add(GF4.one, GF4.one), GF4.x)
    gf4_add(GF4.one, gf4_add(GF4.one, GF4.x_plus_one)) = gf4_add(gf4_add(GF4.one, GF4.one), GF4.x_plus_one)
    w(GF4.zero)
    w(GF4.one)
    w(GF4.x)
    w(GF4.x_plus_one)
    GF4.induction(w)
    w(z)
    gf4_add(GF4.one, gf4_add(GF4.one, z)) = gf4_add(gf4_add(GF4.one, GF4.one), z)
}

/// gf4_add_assoc with the first two arguments fixed.
theorem gf4_add_assoc_e1e2(z: GF4) {
    gf4_add(GF4.one, gf4_add(GF4.x, z)) = gf4_add(gf4_add(GF4.one, GF4.x), z)
} by {
    define w(a: GF4) -> Bool {
        gf4_add(GF4.one, gf4_add(GF4.x, a)) = gf4_add(gf4_add(GF4.one, GF4.x), a)
    }
    gf4_add(GF4.one, gf4_add(GF4.x, GF4.zero)) = gf4_add(gf4_add(GF4.one, GF4.x), GF4.zero)
    gf4_add(GF4.one, gf4_add(GF4.x, GF4.one)) = gf4_add(gf4_add(GF4.one, GF4.x), GF4.one)
    gf4_add(GF4.one, gf4_add(GF4.x, GF4.x)) = gf4_add(gf4_add(GF4.one, GF4.x), GF4.x)
    gf4_add(GF4.one, gf4_add(GF4.x, GF4.x_plus_one)) = gf4_add(gf4_add(GF4.one, GF4.x), GF4.x_plus_one)
    w(GF4.zero)
    w(GF4.one)
    w(GF4.x)
    w(GF4.x_plus_one)
    GF4.induction(w)
    w(z)
    gf4_add(GF4.one, gf4_add(GF4.x, z)) = gf4_add(gf4_add(GF4.one, GF4.x), z)
}

/// gf4_add_assoc with the first two arguments fixed.
theorem gf4_add_assoc_e1e3(z: GF4) {
    gf4_add(GF4.one, gf4_add(GF4.x_plus_one, z)) = gf4_add(gf4_add(GF4.one, GF4.x_plus_one), z)
} by {
    define w(a: GF4) -> Bool {
        gf4_add(GF4.one, gf4_add(GF4.x_plus_one, a)) = gf4_add(gf4_add(GF4.one, GF4.x_plus_one), a)
    }
    gf4_add(GF4.one, gf4_add(GF4.x_plus_one, GF4.zero)) = gf4_add(gf4_add(GF4.one, GF4.x_plus_one), GF4.zero)
    gf4_add(GF4.one, gf4_add(GF4.x_plus_one, GF4.one)) = gf4_add(gf4_add(GF4.one, GF4.x_plus_one), GF4.one)
    gf4_add(GF4.one, gf4_add(GF4.x_plus_one, GF4.x)) = gf4_add(gf4_add(GF4.one, GF4.x_plus_one), GF4.x)
    gf4_add(GF4.one, gf4_add(GF4.x_plus_one, GF4.x_plus_one)) = gf4_add(gf4_add(GF4.one, GF4.x_plus_one), GF4.x_plus_one)
    w(GF4.zero)
    w(GF4.one)
    w(GF4.x)
    w(GF4.x_plus_one)
    GF4.induction(w)
    w(z)
    gf4_add(GF4.one, gf4_add(GF4.x_plus_one, z)) = gf4_add(gf4_add(GF4.one, GF4.x_plus_one), z)
}

/// gf4_add_assoc with the first two arguments fixed.
theorem gf4_add_assoc_e2e0(z: GF4) {
    gf4_add(GF4.x, gf4_add(GF4.zero, z)) = gf4_add(gf4_add(GF4.x, GF4.zero), z)
} by {
    define w(a: GF4) -> Bool {
        gf4_add(GF4.x, gf4_add(GF4.zero, a)) = gf4_add(gf4_add(GF4.x, GF4.zero), a)
    }
    gf4_add(GF4.x, gf4_add(GF4.zero, GF4.zero)) = gf4_add(gf4_add(GF4.x, GF4.zero), GF4.zero)
    gf4_add(GF4.x, gf4_add(GF4.zero, GF4.one)) = gf4_add(gf4_add(GF4.x, GF4.zero), GF4.one)
    gf4_add(GF4.x, gf4_add(GF4.zero, GF4.x)) = gf4_add(gf4_add(GF4.x, GF4.zero), GF4.x)
    gf4_add(GF4.x, gf4_add(GF4.zero, GF4.x_plus_one)) = gf4_add(gf4_add(GF4.x, GF4.zero), GF4.x_plus_one)
    w(GF4.zero)
    w(GF4.one)
    w(GF4.x)
    w(GF4.x_plus_one)
    GF4.induction(w)
    w(z)
    gf4_add(GF4.x, gf4_add(GF4.zero, z)) = gf4_add(gf4_add(GF4.x, GF4.zero), z)
}

/// gf4_add_assoc with the first two arguments fixed.
theorem gf4_add_assoc_e2e1(z: GF4) {
    gf4_add(GF4.x, gf4_add(GF4.one, z)) = gf4_add(gf4_add(GF4.x, GF4.one), z)
} by {
    define w(a: GF4) -> Bool {
        gf4_add(GF4.x, gf4_add(GF4.one, a)) = gf4_add(gf4_add(GF4.x, GF4.one), a)
    }
    gf4_add(GF4.x, gf4_add(GF4.one, GF4.zero)) = gf4_add(gf4_add(GF4.x, GF4.one), GF4.zero)
    gf4_add(GF4.x, gf4_add(GF4.one, GF4.one)) = gf4_add(gf4_add(GF4.x, GF4.one), GF4.one)
    gf4_add(GF4.x, gf4_add(GF4.one, GF4.x)) = gf4_add(gf4_add(GF4.x, GF4.one), GF4.x)
    gf4_add(GF4.x, gf4_add(GF4.one, GF4.x_plus_one)) = gf4_add(gf4_add(GF4.x, GF4.one), GF4.x_plus_one)
    w(GF4.zero)
    w(GF4.one)
    w(GF4.x)
    w(GF4.x_plus_one)
    GF4.induction(w)
    w(z)
    gf4_add(GF4.x, gf4_add(GF4.one, z)) = gf4_add(gf4_add(GF4.x, GF4.one), z)
}

/// gf4_add_assoc with the first two arguments fixed.
theorem gf4_add_assoc_e2e2(z: GF4) {
    gf4_add(GF4.x, gf4_add(GF4.x, z)) = gf4_add(gf4_add(GF4.x, GF4.x), z)
} by {
    define w(a: GF4) -> Bool {
        gf4_add(GF4.x, gf4_add(GF4.x, a)) = gf4_add(gf4_add(GF4.x, GF4.x), a)
    }
    gf4_add(GF4.x, gf4_add(GF4.x, GF4.zero)) = gf4_add(gf4_add(GF4.x, GF4.x), GF4.zero)
    gf4_add(GF4.x, gf4_add(GF4.x, GF4.one)) = gf4_add(gf4_add(GF4.x, GF4.x), GF4.one)
    gf4_add(GF4.x, gf4_add(GF4.x, GF4.x)) = gf4_add(gf4_add(GF4.x, GF4.x), GF4.x)
    gf4_add(GF4.x, gf4_add(GF4.x, GF4.x_plus_one)) = gf4_add(gf4_add(GF4.x, GF4.x), GF4.x_plus_one)
    w(GF4.zero)
    w(GF4.one)
    w(GF4.x)
    w(GF4.x_plus_one)
    GF4.induction(w)
    w(z)
    gf4_add(GF4.x, gf4_add(GF4.x, z)) = gf4_add(gf4_add(GF4.x, GF4.x), z)
}

/// gf4_add_assoc with the first two arguments fixed.
theorem gf4_add_assoc_e2e3(z: GF4) {
    gf4_add(GF4.x, gf4_add(GF4.x_plus_one, z)) = gf4_add(gf4_add(GF4.x, GF4.x_plus_one), z)
} by {
    define w(a: GF4) -> Bool {
        gf4_add(GF4.x, gf4_add(GF4.x_plus_one, a)) = gf4_add(gf4_add(GF4.x, GF4.x_plus_one), a)
    }
    gf4_add(GF4.x, gf4_add(GF4.x_plus_one, GF4.zero)) = gf4_add(gf4_add(GF4.x, GF4.x_plus_one), GF4.zero)
    gf4_add(GF4.x, gf4_add(GF4.x_plus_one, GF4.one)) = gf4_add(gf4_add(GF4.x, GF4.x_plus_one), GF4.one)
    gf4_add(GF4.x, gf4_add(GF4.x_plus_one, GF4.x)) = gf4_add(gf4_add(GF4.x, GF4.x_plus_one), GF4.x)
    gf4_add(GF4.x, gf4_add(GF4.x_plus_one, GF4.x_plus_one)) = gf4_add(gf4_add(GF4.x, GF4.x_plus_one), GF4.x_plus_one)
    w(GF4.zero)
    w(GF4.one)
    w(GF4.x)
    w(GF4.x_plus_one)
    GF4.induction(w)
    w(z)
    gf4_add(GF4.x, gf4_add(GF4.x_plus_one, z)) = gf4_add(gf4_add(GF4.x, GF4.x_plus_one), z)
}

/// gf4_add_assoc with the first two arguments fixed.
theorem gf4_add_assoc_e3e0(z: GF4) {
    gf4_add(GF4.x_plus_one, gf4_add(GF4.zero, z)) = gf4_add(gf4_add(GF4.x_plus_one, GF4.zero), z)
} by {
    define w(a: GF4) -> Bool {
        gf4_add(GF4.x_plus_one, gf4_add(GF4.zero, a)) = gf4_add(gf4_add(GF4.x_plus_one, GF4.zero), a)
    }
    gf4_add(GF4.x_plus_one, gf4_add(GF4.zero, GF4.zero)) = gf4_add(gf4_add(GF4.x_plus_one, GF4.zero), GF4.zero)
    gf4_add(GF4.x_plus_one, gf4_add(GF4.zero, GF4.one)) = gf4_add(gf4_add(GF4.x_plus_one, GF4.zero), GF4.one)
    gf4_add(GF4.x_plus_one, gf4_add(GF4.zero, GF4.x)) = gf4_add(gf4_add(GF4.x_plus_one, GF4.zero), GF4.x)
    gf4_add(GF4.x_plus_one, gf4_add(GF4.zero, GF4.x_plus_one)) = gf4_add(gf4_add(GF4.x_plus_one, GF4.zero), GF4.x_plus_one)
    w(GF4.zero)
    w(GF4.one)
    w(GF4.x)
    w(GF4.x_plus_one)
    GF4.induction(w)
    w(z)
    gf4_add(GF4.x_plus_one, gf4_add(GF4.zero, z)) = gf4_add(gf4_add(GF4.x_plus_one, GF4.zero), z)
}

/// gf4_add_assoc with the first two arguments fixed.
theorem gf4_add_assoc_e3e1(z: GF4) {
    gf4_add(GF4.x_plus_one, gf4_add(GF4.one, z)) = gf4_add(gf4_add(GF4.x_plus_one, GF4.one), z)
} by {
    define w(a: GF4) -> Bool {
        gf4_add(GF4.x_plus_one, gf4_add(GF4.one, a)) = gf4_add(gf4_add(GF4.x_plus_one, GF4.one), a)
    }
    gf4_add(GF4.x_plus_one, gf4_add(GF4.one, GF4.zero)) = gf4_add(gf4_add(GF4.x_plus_one, GF4.one), GF4.zero)
    gf4_add(GF4.x_plus_one, gf4_add(GF4.one, GF4.one)) = gf4_add(gf4_add(GF4.x_plus_one, GF4.one), GF4.one)
    gf4_add(GF4.x_plus_one, gf4_add(GF4.one, GF4.x)) = gf4_add(gf4_add(GF4.x_plus_one, GF4.one), GF4.x)
    gf4_add(GF4.x_plus_one, gf4_add(GF4.one, GF4.x_plus_one)) = gf4_add(gf4_add(GF4.x_plus_one, GF4.one), GF4.x_plus_one)
    w(GF4.zero)
    w(GF4.one)
    w(GF4.x)
    w(GF4.x_plus_one)
    GF4.induction(w)
    w(z)
    gf4_add(GF4.x_plus_one, gf4_add(GF4.one, z)) = gf4_add(gf4_add(GF4.x_plus_one, GF4.one), z)
}

/// gf4_add_assoc with the first two arguments fixed.
theorem gf4_add_assoc_e3e2(z: GF4) {
    gf4_add(GF4.x_plus_one, gf4_add(GF4.x, z)) = gf4_add(gf4_add(GF4.x_plus_one, GF4.x), z)
} by {
    define w(a: GF4) -> Bool {
        gf4_add(GF4.x_plus_one, gf4_add(GF4.x, a)) = gf4_add(gf4_add(GF4.x_plus_one, GF4.x), a)
    }
    gf4_add(GF4.x_plus_one, gf4_add(GF4.x, GF4.zero)) = gf4_add(gf4_add(GF4.x_plus_one, GF4.x), GF4.zero)
    gf4_add(GF4.x_plus_one, gf4_add(GF4.x, GF4.one)) = gf4_add(gf4_add(GF4.x_plus_one, GF4.x), GF4.one)
    gf4_add(GF4.x_plus_one, gf4_add(GF4.x, GF4.x)) = gf4_add(gf4_add(GF4.x_plus_one, GF4.x), GF4.x)
    gf4_add(GF4.x_plus_one, gf4_add(GF4.x, GF4.x_plus_one)) = gf4_add(gf4_add(GF4.x_plus_one, GF4.x), GF4.x_plus_one)
    w(GF4.zero)
    w(GF4.one)
    w(GF4.x)
    w(GF4.x_plus_one)
    GF4.induction(w)
    w(z)
    gf4_add(GF4.x_plus_one, gf4_add(GF4.x, z)) = gf4_add(gf4_add(GF4.x_plus_one, GF4.x), z)
}

/// gf4_add_assoc with the first two arguments fixed.
theorem gf4_add_assoc_e3e3(z: GF4) {
    gf4_add(GF4.x_plus_one, gf4_add(GF4.x_plus_one, z)) = gf4_add(gf4_add(GF4.x_plus_one, GF4.x_plus_one), z)
} by {
    define w(a: GF4) -> Bool {
        gf4_add(GF4.x_plus_one, gf4_add(GF4.x_plus_one, a)) = gf4_add(gf4_add(GF4.x_plus_one, GF4.x_plus_one), a)
    }
    gf4_add(GF4.x_plus_one, gf4_add(GF4.x_plus_one, GF4.zero)) = gf4_add(gf4_add(GF4.x_plus_one, GF4.x_plus_one), GF4.zero)
    gf4_add(GF4.x_plus_one, gf4_add(GF4.x_plus_one, GF4.one)) = gf4_add(gf4_add(GF4.x_plus_one, GF4.x_plus_one), GF4.one)
    gf4_add(GF4.x_plus_one, gf4_add(GF4.x_plus_one, GF4.x)) = gf4_add(gf4_add(GF4.x_plus_one, GF4.x_plus_one), GF4.x)
    gf4_add(GF4.x_plus_one, gf4_add(GF4.x_plus_one, GF4.x_plus_one)) = gf4_add(gf4_add(GF4.x_plus_one, GF4.x_plus_one), GF4.x_plus_one)
    w(GF4.zero)
    w(GF4.one)
    w(GF4.x)
    w(GF4.x_plus_one)
    GF4.induction(w)
    w(z)
    gf4_add(GF4.x_plus_one, gf4_add(GF4.x_plus_one, z)) = gf4_add(gf4_add(GF4.x_plus_one, GF4.x_plus_one), z)
}

/// gf4_add_assoc with the first argument fixed at GF4.zero.
theorem gf4_add_assoc_e0(y: GF4, z: GF4) {
    gf4_add(GF4.zero, gf4_add(y, z)) = gf4_add(gf4_add(GF4.zero, y), z)
} by {
    gf4_cases(y)
    if y = GF4.zero {
        gf4_add_assoc_e0e0(z)
        gf4_add(GF4.zero, gf4_add(y, z)) = gf4_add(gf4_add(GF4.zero, y), z)
    }
    if y = GF4.one {
        gf4_add_assoc_e0e1(z)
        gf4_add(GF4.zero, gf4_add(y, z)) = gf4_add(gf4_add(GF4.zero, y), z)
    }
    if y = GF4.x {
        gf4_add_assoc_e0e2(z)
        gf4_add(GF4.zero, gf4_add(y, z)) = gf4_add(gf4_add(GF4.zero, y), z)
    }
    if y = GF4.x_plus_one {
        gf4_add_assoc_e0e3(z)
        gf4_add(GF4.zero, gf4_add(y, z)) = gf4_add(gf4_add(GF4.zero, y), z)
    }
    gf4_add(GF4.zero, gf4_add(y, z)) = gf4_add(gf4_add(GF4.zero, y), z)
}

/// gf4_add_assoc with the first argument fixed at GF4.one.
theorem gf4_add_assoc_e1(y: GF4, z: GF4) {
    gf4_add(GF4.one, gf4_add(y, z)) = gf4_add(gf4_add(GF4.one, y), z)
} by {
    gf4_cases(y)
    if y = GF4.zero {
        gf4_add_assoc_e1e0(z)
        gf4_add(GF4.one, gf4_add(y, z)) = gf4_add(gf4_add(GF4.one, y), z)
    }
    if y = GF4.one {
        gf4_add_assoc_e1e1(z)
        gf4_add(GF4.one, gf4_add(y, z)) = gf4_add(gf4_add(GF4.one, y), z)
    }
    if y = GF4.x {
        gf4_add_assoc_e1e2(z)
        gf4_add(GF4.one, gf4_add(y, z)) = gf4_add(gf4_add(GF4.one, y), z)
    }
    if y = GF4.x_plus_one {
        gf4_add_assoc_e1e3(z)
        gf4_add(GF4.one, gf4_add(y, z)) = gf4_add(gf4_add(GF4.one, y), z)
    }
    gf4_add(GF4.one, gf4_add(y, z)) = gf4_add(gf4_add(GF4.one, y), z)
}

/// gf4_add_assoc with the first argument fixed at GF4.x.
theorem gf4_add_assoc_e2(y: GF4, z: GF4) {
    gf4_add(GF4.x, gf4_add(y, z)) = gf4_add(gf4_add(GF4.x, y), z)
} by {
    gf4_cases(y)
    if y = GF4.zero {
        gf4_add_assoc_e2e0(z)
        gf4_add(GF4.x, gf4_add(y, z)) = gf4_add(gf4_add(GF4.x, y), z)
    }
    if y = GF4.one {
        gf4_add_assoc_e2e1(z)
        gf4_add(GF4.x, gf4_add(y, z)) = gf4_add(gf4_add(GF4.x, y), z)
    }
    if y = GF4.x {
        gf4_add_assoc_e2e2(z)
        gf4_add(GF4.x, gf4_add(y, z)) = gf4_add(gf4_add(GF4.x, y), z)
    }
    if y = GF4.x_plus_one {
        gf4_add_assoc_e2e3(z)
        gf4_add(GF4.x, gf4_add(y, z)) = gf4_add(gf4_add(GF4.x, y), z)
    }
    gf4_add(GF4.x, gf4_add(y, z)) = gf4_add(gf4_add(GF4.x, y), z)
}

/// gf4_add_assoc with the first argument fixed at GF4.x_plus_one.
theorem gf4_add_assoc_e3(y: GF4, z: GF4) {
    gf4_add(GF4.x_plus_one, gf4_add(y, z)) = gf4_add(gf4_add(GF4.x_plus_one, y), z)
} by {
    gf4_cases(y)
    if y = GF4.zero {
        gf4_add_assoc_e3e0(z)
        gf4_add(GF4.x_plus_one, gf4_add(y, z)) = gf4_add(gf4_add(GF4.x_plus_one, y), z)
    }
    if y = GF4.one {
        gf4_add_assoc_e3e1(z)
        gf4_add(GF4.x_plus_one, gf4_add(y, z)) = gf4_add(gf4_add(GF4.x_plus_one, y), z)
    }
    if y = GF4.x {
        gf4_add_assoc_e3e2(z)
        gf4_add(GF4.x_plus_one, gf4_add(y, z)) = gf4_add(gf4_add(GF4.x_plus_one, y), z)
    }
    if y = GF4.x_plus_one {
        gf4_add_assoc_e3e3(z)
        gf4_add(GF4.x_plus_one, gf4_add(y, z)) = gf4_add(gf4_add(GF4.x_plus_one, y), z)
    }
    gf4_add(GF4.x_plus_one, gf4_add(y, z)) = gf4_add(gf4_add(GF4.x_plus_one, y), z)
}

/// gf4_add_assoc.
theorem gf4_add_assoc(x: GF4, y: GF4, z: GF4) {
    Add.add(x, Add.add(y, z)) = Add.add(Add.add(x, y), z)
} by {
    gf4_cases(x)
    if x = GF4.zero {
        gf4_add_assoc_e0(y, z)
        gf4_add(GF4.zero, gf4_add(y, z)) = gf4_add(gf4_add(GF4.zero, y), z)
        gf4_add(x, gf4_add(y, z)) = gf4_add(gf4_add(x, y), z)
    }
    if x = GF4.one {
        gf4_add_assoc_e1(y, z)
        gf4_add(GF4.one, gf4_add(y, z)) = gf4_add(gf4_add(GF4.one, y), z)
        gf4_add(x, gf4_add(y, z)) = gf4_add(gf4_add(x, y), z)
    }
    if x = GF4.x {
        gf4_add_assoc_e2(y, z)
        gf4_add(GF4.x, gf4_add(y, z)) = gf4_add(gf4_add(GF4.x, y), z)
        gf4_add(x, gf4_add(y, z)) = gf4_add(gf4_add(x, y), z)
    }
    if x = GF4.x_plus_one {
        gf4_add_assoc_e3(y, z)
        gf4_add(GF4.x_plus_one, gf4_add(y, z)) = gf4_add(gf4_add(GF4.x_plus_one, y), z)
        gf4_add(x, gf4_add(y, z)) = gf4_add(gf4_add(x, y), z)
    }
    gf4_add(x, gf4_add(y, z)) = gf4_add(gf4_add(x, y), z)
    Add.add(x, Add.add(y, z)) = Add.add(Add.add(x, y), z)
}

/// gf4_mul_assoc with the first two arguments fixed.
theorem gf4_mul_assoc_e0e0(z: GF4) {
    gf4_mul(GF4.zero, gf4_mul(GF4.zero, z)) = gf4_mul(gf4_mul(GF4.zero, GF4.zero), z)
} by {
    define w(a: GF4) -> Bool {
        gf4_mul(GF4.zero, gf4_mul(GF4.zero, a)) = gf4_mul(gf4_mul(GF4.zero, GF4.zero), a)
    }
    gf4_mul(GF4.zero, gf4_mul(GF4.zero, GF4.zero)) = gf4_mul(gf4_mul(GF4.zero, GF4.zero), GF4.zero)
    gf4_mul(GF4.zero, gf4_mul(GF4.zero, GF4.one)) = gf4_mul(gf4_mul(GF4.zero, GF4.zero), GF4.one)
    gf4_mul(GF4.zero, gf4_mul(GF4.zero, GF4.x)) = gf4_mul(gf4_mul(GF4.zero, GF4.zero), GF4.x)
    gf4_mul(GF4.zero, gf4_mul(GF4.zero, GF4.x_plus_one)) = gf4_mul(gf4_mul(GF4.zero, GF4.zero), GF4.x_plus_one)
    w(GF4.zero)
    w(GF4.one)
    w(GF4.x)
    w(GF4.x_plus_one)
    GF4.induction(w)
    w(z)
    gf4_mul(GF4.zero, gf4_mul(GF4.zero, z)) = gf4_mul(gf4_mul(GF4.zero, GF4.zero), z)
}

/// gf4_mul_assoc with the first two arguments fixed.
theorem gf4_mul_assoc_e0e1(z: GF4) {
    gf4_mul(GF4.zero, gf4_mul(GF4.one, z)) = gf4_mul(gf4_mul(GF4.zero, GF4.one), z)
} by {
    define w(a: GF4) -> Bool {
        gf4_mul(GF4.zero, gf4_mul(GF4.one, a)) = gf4_mul(gf4_mul(GF4.zero, GF4.one), a)
    }
    gf4_mul(GF4.zero, gf4_mul(GF4.one, GF4.zero)) = gf4_mul(gf4_mul(GF4.zero, GF4.one), GF4.zero)
    gf4_mul(GF4.zero, gf4_mul(GF4.one, GF4.one)) = gf4_mul(gf4_mul(GF4.zero, GF4.one), GF4.one)
    gf4_mul(GF4.zero, gf4_mul(GF4.one, GF4.x)) = gf4_mul(gf4_mul(GF4.zero, GF4.one), GF4.x)
    gf4_mul(GF4.zero, gf4_mul(GF4.one, GF4.x_plus_one)) = gf4_mul(gf4_mul(GF4.zero, GF4.one), GF4.x_plus_one)
    w(GF4.zero)
    w(GF4.one)
    w(GF4.x)
    w(GF4.x_plus_one)
    GF4.induction(w)
    w(z)
    gf4_mul(GF4.zero, gf4_mul(GF4.one, z)) = gf4_mul(gf4_mul(GF4.zero, GF4.one), z)
}

/// gf4_mul_assoc with the first two arguments fixed.
theorem gf4_mul_assoc_e0e2(z: GF4) {
    gf4_mul(GF4.zero, gf4_mul(GF4.x, z)) = gf4_mul(gf4_mul(GF4.zero, GF4.x), z)
} by {
    define w(a: GF4) -> Bool {
        gf4_mul(GF4.zero, gf4_mul(GF4.x, a)) = gf4_mul(gf4_mul(GF4.zero, GF4.x), a)
    }
    gf4_mul(GF4.zero, gf4_mul(GF4.x, GF4.zero)) = gf4_mul(gf4_mul(GF4.zero, GF4.x), GF4.zero)
    gf4_mul(GF4.zero, gf4_mul(GF4.x, GF4.one)) = gf4_mul(gf4_mul(GF4.zero, GF4.x), GF4.one)
    gf4_mul(GF4.zero, gf4_mul(GF4.x, GF4.x)) = gf4_mul(gf4_mul(GF4.zero, GF4.x), GF4.x)
    gf4_mul(GF4.zero, gf4_mul(GF4.x, GF4.x_plus_one)) = gf4_mul(gf4_mul(GF4.zero, GF4.x), GF4.x_plus_one)
    w(GF4.zero)
    w(GF4.one)
    w(GF4.x)
    w(GF4.x_plus_one)
    GF4.induction(w)
    w(z)
    gf4_mul(GF4.zero, gf4_mul(GF4.x, z)) = gf4_mul(gf4_mul(GF4.zero, GF4.x), z)
}

/// gf4_mul_assoc with the first two arguments fixed.
theorem gf4_mul_assoc_e0e3(z: GF4) {
    gf4_mul(GF4.zero, gf4_mul(GF4.x_plus_one, z)) = gf4_mul(gf4_mul(GF4.zero, GF4.x_plus_one), z)
} by {
    define w(a: GF4) -> Bool {
        gf4_mul(GF4.zero, gf4_mul(GF4.x_plus_one, a)) = gf4_mul(gf4_mul(GF4.zero, GF4.x_plus_one), a)
    }
    gf4_mul(GF4.zero, gf4_mul(GF4.x_plus_one, GF4.zero)) = gf4_mul(gf4_mul(GF4.zero, GF4.x_plus_one), GF4.zero)
    gf4_mul(GF4.zero, gf4_mul(GF4.x_plus_one, GF4.one)) = gf4_mul(gf4_mul(GF4.zero, GF4.x_plus_one), GF4.one)
    gf4_mul(GF4.zero, gf4_mul(GF4.x_plus_one, GF4.x)) = gf4_mul(gf4_mul(GF4.zero, GF4.x_plus_one), GF4.x)
    gf4_mul(GF4.zero, gf4_mul(GF4.x_plus_one, GF4.x_plus_one)) = gf4_mul(gf4_mul(GF4.zero, GF4.x_plus_one), GF4.x_plus_one)
    w(GF4.zero)
    w(GF4.one)
    w(GF4.x)
    w(GF4.x_plus_one)
    GF4.induction(w)
    w(z)
    gf4_mul(GF4.zero, gf4_mul(GF4.x_plus_one, z)) = gf4_mul(gf4_mul(GF4.zero, GF4.x_plus_one), z)
}

/// gf4_mul_assoc with the first two arguments fixed.
theorem gf4_mul_assoc_e1e0(z: GF4) {
    gf4_mul(GF4.one, gf4_mul(GF4.zero, z)) = gf4_mul(gf4_mul(GF4.one, GF4.zero), z)
} by {
    define w(a: GF4) -> Bool {
        gf4_mul(GF4.one, gf4_mul(GF4.zero, a)) = gf4_mul(gf4_mul(GF4.one, GF4.zero), a)
    }
    gf4_mul(GF4.one, gf4_mul(GF4.zero, GF4.zero)) = gf4_mul(gf4_mul(GF4.one, GF4.zero), GF4.zero)
    gf4_mul(GF4.one, gf4_mul(GF4.zero, GF4.one)) = gf4_mul(gf4_mul(GF4.one, GF4.zero), GF4.one)
    gf4_mul(GF4.one, gf4_mul(GF4.zero, GF4.x)) = gf4_mul(gf4_mul(GF4.one, GF4.zero), GF4.x)
    gf4_mul(GF4.one, gf4_mul(GF4.zero, GF4.x_plus_one)) = gf4_mul(gf4_mul(GF4.one, GF4.zero), GF4.x_plus_one)
    w(GF4.zero)
    w(GF4.one)
    w(GF4.x)
    w(GF4.x_plus_one)
    GF4.induction(w)
    w(z)
    gf4_mul(GF4.one, gf4_mul(GF4.zero, z)) = gf4_mul(gf4_mul(GF4.one, GF4.zero), z)
}

/// gf4_mul_assoc with the first two arguments fixed.
theorem gf4_mul_assoc_e1e1(z: GF4) {
    gf4_mul(GF4.one, gf4_mul(GF4.one, z)) = gf4_mul(gf4_mul(GF4.one, GF4.one), z)
} by {
    define w(a: GF4) -> Bool {
        gf4_mul(GF4.one, gf4_mul(GF4.one, a)) = gf4_mul(gf4_mul(GF4.one, GF4.one), a)
    }
    gf4_mul(GF4.one, gf4_mul(GF4.one, GF4.zero)) = gf4_mul(gf4_mul(GF4.one, GF4.one), GF4.zero)
    gf4_mul(GF4.one, gf4_mul(GF4.one, GF4.one)) = gf4_mul(gf4_mul(GF4.one, GF4.one), GF4.one)
    gf4_mul(GF4.one, gf4_mul(GF4.one, GF4.x)) = gf4_mul(gf4_mul(GF4.one, GF4.one), GF4.x)
    gf4_mul(GF4.one, gf4_mul(GF4.one, GF4.x_plus_one)) = gf4_mul(gf4_mul(GF4.one, GF4.one), GF4.x_plus_one)
    w(GF4.zero)
    w(GF4.one)
    w(GF4.x)
    w(GF4.x_plus_one)
    GF4.induction(w)
    w(z)
    gf4_mul(GF4.one, gf4_mul(GF4.one, z)) = gf4_mul(gf4_mul(GF4.one, GF4.one), z)
}

/// gf4_mul_assoc with the first two arguments fixed.
theorem gf4_mul_assoc_e1e2(z: GF4) {
    gf4_mul(GF4.one, gf4_mul(GF4.x, z)) = gf4_mul(gf4_mul(GF4.one, GF4.x), z)
} by {
    define w(a: GF4) -> Bool {
        gf4_mul(GF4.one, gf4_mul(GF4.x, a)) = gf4_mul(gf4_mul(GF4.one, GF4.x), a)
    }
    gf4_mul(GF4.one, gf4_mul(GF4.x, GF4.zero)) = gf4_mul(gf4_mul(GF4.one, GF4.x), GF4.zero)
    gf4_mul(GF4.one, gf4_mul(GF4.x, GF4.one)) = gf4_mul(gf4_mul(GF4.one, GF4.x), GF4.one)
    gf4_mul(GF4.one, gf4_mul(GF4.x, GF4.x)) = gf4_mul(gf4_mul(GF4.one, GF4.x), GF4.x)
    gf4_mul(GF4.one, gf4_mul(GF4.x, GF4.x_plus_one)) = gf4_mul(gf4_mul(GF4.one, GF4.x), GF4.x_plus_one)
    w(GF4.zero)
    w(GF4.one)
    w(GF4.x)
    w(GF4.x_plus_one)
    GF4.induction(w)
    w(z)
    gf4_mul(GF4.one, gf4_mul(GF4.x, z)) = gf4_mul(gf4_mul(GF4.one, GF4.x), z)
}

/// gf4_mul_assoc with the first two arguments fixed.
theorem gf4_mul_assoc_e1e3(z: GF4) {
    gf4_mul(GF4.one, gf4_mul(GF4.x_plus_one, z)) = gf4_mul(gf4_mul(GF4.one, GF4.x_plus_one), z)
} by {
    define w(a: GF4) -> Bool {
        gf4_mul(GF4.one, gf4_mul(GF4.x_plus_one, a)) = gf4_mul(gf4_mul(GF4.one, GF4.x_plus_one), a)
    }
    gf4_mul(GF4.one, gf4_mul(GF4.x_plus_one, GF4.zero)) = gf4_mul(gf4_mul(GF4.one, GF4.x_plus_one), GF4.zero)
    gf4_mul(GF4.one, gf4_mul(GF4.x_plus_one, GF4.one)) = gf4_mul(gf4_mul(GF4.one, GF4.x_plus_one), GF4.one)
    gf4_mul(GF4.one, gf4_mul(GF4.x_plus_one, GF4.x)) = gf4_mul(gf4_mul(GF4.one, GF4.x_plus_one), GF4.x)
    gf4_mul(GF4.one, gf4_mul(GF4.x_plus_one, GF4.x_plus_one)) = gf4_mul(gf4_mul(GF4.one, GF4.x_plus_one), GF4.x_plus_one)
    w(GF4.zero)
    w(GF4.one)
    w(GF4.x)
    w(GF4.x_plus_one)
    GF4.induction(w)
    w(z)
    gf4_mul(GF4.one, gf4_mul(GF4.x_plus_one, z)) = gf4_mul(gf4_mul(GF4.one, GF4.x_plus_one), z)
}

/// gf4_mul_assoc with the first two arguments fixed.
theorem gf4_mul_assoc_e2e0(z: GF4) {
    gf4_mul(GF4.x, gf4_mul(GF4.zero, z)) = gf4_mul(gf4_mul(GF4.x, GF4.zero), z)
} by {
    define w(a: GF4) -> Bool {
        gf4_mul(GF4.x, gf4_mul(GF4.zero, a)) = gf4_mul(gf4_mul(GF4.x, GF4.zero), a)
    }
    gf4_mul(GF4.x, gf4_mul(GF4.zero, GF4.zero)) = gf4_mul(gf4_mul(GF4.x, GF4.zero), GF4.zero)
    gf4_mul(GF4.x, gf4_mul(GF4.zero, GF4.one)) = gf4_mul(gf4_mul(GF4.x, GF4.zero), GF4.one)
    gf4_mul(GF4.x, gf4_mul(GF4.zero, GF4.x)) = gf4_mul(gf4_mul(GF4.x, GF4.zero), GF4.x)
    gf4_mul(GF4.x, gf4_mul(GF4.zero, GF4.x_plus_one)) = gf4_mul(gf4_mul(GF4.x, GF4.zero), GF4.x_plus_one)
    w(GF4.zero)
    w(GF4.one)
    w(GF4.x)
    w(GF4.x_plus_one)
    GF4.induction(w)
    w(z)
    gf4_mul(GF4.x, gf4_mul(GF4.zero, z)) = gf4_mul(gf4_mul(GF4.x, GF4.zero), z)
}

/// gf4_mul_assoc with the first two arguments fixed.
theorem gf4_mul_assoc_e2e1(z: GF4) {
    gf4_mul(GF4.x, gf4_mul(GF4.one, z)) = gf4_mul(gf4_mul(GF4.x, GF4.one), z)
} by {
    define w(a: GF4) -> Bool {
        gf4_mul(GF4.x, gf4_mul(GF4.one, a)) = gf4_mul(gf4_mul(GF4.x, GF4.one), a)
    }
    gf4_mul(GF4.x, gf4_mul(GF4.one, GF4.zero)) = gf4_mul(gf4_mul(GF4.x, GF4.one), GF4.zero)
    gf4_mul(GF4.x, gf4_mul(GF4.one, GF4.one)) = gf4_mul(gf4_mul(GF4.x, GF4.one), GF4.one)
    gf4_mul(GF4.x, gf4_mul(GF4.one, GF4.x)) = gf4_mul(gf4_mul(GF4.x, GF4.one), GF4.x)
    gf4_mul(GF4.x, gf4_mul(GF4.one, GF4.x_plus_one)) = gf4_mul(gf4_mul(GF4.x, GF4.one), GF4.x_plus_one)
    w(GF4.zero)
    w(GF4.one)
    w(GF4.x)
    w(GF4.x_plus_one)
    GF4.induction(w)
    w(z)
    gf4_mul(GF4.x, gf4_mul(GF4.one, z)) = gf4_mul(gf4_mul(GF4.x, GF4.one), z)
}

/// gf4_mul_assoc with the first two arguments fixed.
theorem gf4_mul_assoc_e2e2(z: GF4) {
    gf4_mul(GF4.x, gf4_mul(GF4.x, z)) = gf4_mul(gf4_mul(GF4.x, GF4.x), z)
} by {
    define w(a: GF4) -> Bool {
        gf4_mul(GF4.x, gf4_mul(GF4.x, a)) = gf4_mul(gf4_mul(GF4.x, GF4.x), a)
    }
    gf4_mul(GF4.x, gf4_mul(GF4.x, GF4.zero)) = gf4_mul(gf4_mul(GF4.x, GF4.x), GF4.zero)
    gf4_mul(GF4.x, gf4_mul(GF4.x, GF4.one)) = gf4_mul(gf4_mul(GF4.x, GF4.x), GF4.one)
    gf4_mul(GF4.x, gf4_mul(GF4.x, GF4.x)) = gf4_mul(gf4_mul(GF4.x, GF4.x), GF4.x)
    gf4_mul(GF4.x, gf4_mul(GF4.x, GF4.x_plus_one)) = gf4_mul(gf4_mul(GF4.x, GF4.x), GF4.x_plus_one)
    w(GF4.zero)
    w(GF4.one)
    w(GF4.x)
    w(GF4.x_plus_one)
    GF4.induction(w)
    w(z)
    gf4_mul(GF4.x, gf4_mul(GF4.x, z)) = gf4_mul(gf4_mul(GF4.x, GF4.x), z)
}

/// gf4_mul_assoc with the first two arguments fixed.
theorem gf4_mul_assoc_e2e3(z: GF4) {
    gf4_mul(GF4.x, gf4_mul(GF4.x_plus_one, z)) = gf4_mul(gf4_mul(GF4.x, GF4.x_plus_one), z)
} by {
    define w(a: GF4) -> Bool {
        gf4_mul(GF4.x, gf4_mul(GF4.x_plus_one, a)) = gf4_mul(gf4_mul(GF4.x, GF4.x_plus_one), a)
    }
    gf4_mul(GF4.x, gf4_mul(GF4.x_plus_one, GF4.zero)) = gf4_mul(gf4_mul(GF4.x, GF4.x_plus_one), GF4.zero)
    gf4_mul(GF4.x, gf4_mul(GF4.x_plus_one, GF4.one)) = gf4_mul(gf4_mul(GF4.x, GF4.x_plus_one), GF4.one)
    gf4_mul(GF4.x, gf4_mul(GF4.x_plus_one, GF4.x)) = gf4_mul(gf4_mul(GF4.x, GF4.x_plus_one), GF4.x)
    gf4_mul(GF4.x, gf4_mul(GF4.x_plus_one, GF4.x_plus_one)) = gf4_mul(gf4_mul(GF4.x, GF4.x_plus_one), GF4.x_plus_one)
    w(GF4.zero)
    w(GF4.one)
    w(GF4.x)
    w(GF4.x_plus_one)
    GF4.induction(w)
    w(z)
    gf4_mul(GF4.x, gf4_mul(GF4.x_plus_one, z)) = gf4_mul(gf4_mul(GF4.x, GF4.x_plus_one), z)
}

/// gf4_mul_assoc with the first two arguments fixed.
theorem gf4_mul_assoc_e3e0(z: GF4) {
    gf4_mul(GF4.x_plus_one, gf4_mul(GF4.zero, z)) = gf4_mul(gf4_mul(GF4.x_plus_one, GF4.zero), z)
} by {
    define w(a: GF4) -> Bool {
        gf4_mul(GF4.x_plus_one, gf4_mul(GF4.zero, a)) = gf4_mul(gf4_mul(GF4.x_plus_one, GF4.zero), a)
    }
    gf4_mul(GF4.x_plus_one, gf4_mul(GF4.zero, GF4.zero)) = gf4_mul(gf4_mul(GF4.x_plus_one, GF4.zero), GF4.zero)
    gf4_mul(GF4.x_plus_one, gf4_mul(GF4.zero, GF4.one)) = gf4_mul(gf4_mul(GF4.x_plus_one, GF4.zero), GF4.one)
    gf4_mul(GF4.x_plus_one, gf4_mul(GF4.zero, GF4.x)) = gf4_mul(gf4_mul(GF4.x_plus_one, GF4.zero), GF4.x)
    gf4_mul(GF4.x_plus_one, gf4_mul(GF4.zero, GF4.x_plus_one)) = gf4_mul(gf4_mul(GF4.x_plus_one, GF4.zero), GF4.x_plus_one)
    w(GF4.zero)
    w(GF4.one)
    w(GF4.x)
    w(GF4.x_plus_one)
    GF4.induction(w)
    w(z)
    gf4_mul(GF4.x_plus_one, gf4_mul(GF4.zero, z)) = gf4_mul(gf4_mul(GF4.x_plus_one, GF4.zero), z)
}

/// gf4_mul_assoc with the first two arguments fixed.
theorem gf4_mul_assoc_e3e1(z: GF4) {
    gf4_mul(GF4.x_plus_one, gf4_mul(GF4.one, z)) = gf4_mul(gf4_mul(GF4.x_plus_one, GF4.one), z)
} by {
    define w(a: GF4) -> Bool {
        gf4_mul(GF4.x_plus_one, gf4_mul(GF4.one, a)) = gf4_mul(gf4_mul(GF4.x_plus_one, GF4.one), a)
    }
    gf4_mul(GF4.x_plus_one, gf4_mul(GF4.one, GF4.zero)) = gf4_mul(gf4_mul(GF4.x_plus_one, GF4.one), GF4.zero)
    gf4_mul(GF4.x_plus_one, gf4_mul(GF4.one, GF4.one)) = gf4_mul(gf4_mul(GF4.x_plus_one, GF4.one), GF4.one)
    gf4_mul(GF4.x_plus_one, gf4_mul(GF4.one, GF4.x)) = gf4_mul(gf4_mul(GF4.x_plus_one, GF4.one), GF4.x)
    gf4_mul(GF4.x_plus_one, gf4_mul(GF4.one, GF4.x_plus_one)) = gf4_mul(gf4_mul(GF4.x_plus_one, GF4.one), GF4.x_plus_one)
    w(GF4.zero)
    w(GF4.one)
    w(GF4.x)
    w(GF4.x_plus_one)
    GF4.induction(w)
    w(z)
    gf4_mul(GF4.x_plus_one, gf4_mul(GF4.one, z)) = gf4_mul(gf4_mul(GF4.x_plus_one, GF4.one), z)
}

/// gf4_mul_assoc with the first two arguments fixed.
theorem gf4_mul_assoc_e3e2(z: GF4) {
    gf4_mul(GF4.x_plus_one, gf4_mul(GF4.x, z)) = gf4_mul(gf4_mul(GF4.x_plus_one, GF4.x), z)
} by {
    define w(a: GF4) -> Bool {
        gf4_mul(GF4.x_plus_one, gf4_mul(GF4.x, a)) = gf4_mul(gf4_mul(GF4.x_plus_one, GF4.x), a)
    }
    gf4_mul(GF4.x_plus_one, gf4_mul(GF4.x, GF4.zero)) = gf4_mul(gf4_mul(GF4.x_plus_one, GF4.x), GF4.zero)
    gf4_mul(GF4.x_plus_one, gf4_mul(GF4.x, GF4.one)) = gf4_mul(gf4_mul(GF4.x_plus_one, GF4.x), GF4.one)
    gf4_mul(GF4.x_plus_one, gf4_mul(GF4.x, GF4.x)) = gf4_mul(gf4_mul(GF4.x_plus_one, GF4.x), GF4.x)
    gf4_mul(GF4.x_plus_one, gf4_mul(GF4.x, GF4.x_plus_one)) = gf4_mul(gf4_mul(GF4.x_plus_one, GF4.x), GF4.x_plus_one)
    w(GF4.zero)
    w(GF4.one)
    w(GF4.x)
    w(GF4.x_plus_one)
    GF4.induction(w)
    w(z)
    gf4_mul(GF4.x_plus_one, gf4_mul(GF4.x, z)) = gf4_mul(gf4_mul(GF4.x_plus_one, GF4.x), z)
}

/// gf4_mul_assoc with the first two arguments fixed.
theorem gf4_mul_assoc_e3e3(z: GF4) {
    gf4_mul(GF4.x_plus_one, gf4_mul(GF4.x_plus_one, z)) = gf4_mul(gf4_mul(GF4.x_plus_one, GF4.x_plus_one), z)
} by {
    define w(a: GF4) -> Bool {
        gf4_mul(GF4.x_plus_one, gf4_mul(GF4.x_plus_one, a)) = gf4_mul(gf4_mul(GF4.x_plus_one, GF4.x_plus_one), a)
    }
    gf4_mul(GF4.x_plus_one, gf4_mul(GF4.x_plus_one, GF4.zero)) = gf4_mul(gf4_mul(GF4.x_plus_one, GF4.x_plus_one), GF4.zero)
    gf4_mul(GF4.x_plus_one, gf4_mul(GF4.x_plus_one, GF4.one)) = gf4_mul(gf4_mul(GF4.x_plus_one, GF4.x_plus_one), GF4.one)
    gf4_mul(GF4.x_plus_one, gf4_mul(GF4.x_plus_one, GF4.x)) = gf4_mul(gf4_mul(GF4.x_plus_one, GF4.x_plus_one), GF4.x)
    gf4_mul(GF4.x_plus_one, gf4_mul(GF4.x_plus_one, GF4.x_plus_one)) = gf4_mul(gf4_mul(GF4.x_plus_one, GF4.x_plus_one), GF4.x_plus_one)
    w(GF4.zero)
    w(GF4.one)
    w(GF4.x)
    w(GF4.x_plus_one)
    GF4.induction(w)
    w(z)
    gf4_mul(GF4.x_plus_one, gf4_mul(GF4.x_plus_one, z)) = gf4_mul(gf4_mul(GF4.x_plus_one, GF4.x_plus_one), z)
}

/// gf4_mul_assoc with the first argument fixed at GF4.zero.
theorem gf4_mul_assoc_e0(y: GF4, z: GF4) {
    gf4_mul(GF4.zero, gf4_mul(y, z)) = gf4_mul(gf4_mul(GF4.zero, y), z)
} by {
    gf4_cases(y)
    if y = GF4.zero {
        gf4_mul_assoc_e0e0(z)
        gf4_mul(GF4.zero, gf4_mul(y, z)) = gf4_mul(gf4_mul(GF4.zero, y), z)
    }
    if y = GF4.one {
        gf4_mul_assoc_e0e1(z)
        gf4_mul(GF4.zero, gf4_mul(y, z)) = gf4_mul(gf4_mul(GF4.zero, y), z)
    }
    if y = GF4.x {
        gf4_mul_assoc_e0e2(z)
        gf4_mul(GF4.zero, gf4_mul(y, z)) = gf4_mul(gf4_mul(GF4.zero, y), z)
    }
    if y = GF4.x_plus_one {
        gf4_mul_assoc_e0e3(z)
        gf4_mul(GF4.zero, gf4_mul(y, z)) = gf4_mul(gf4_mul(GF4.zero, y), z)
    }
    gf4_mul(GF4.zero, gf4_mul(y, z)) = gf4_mul(gf4_mul(GF4.zero, y), z)
}

/// gf4_mul_assoc with the first argument fixed at GF4.one.
theorem gf4_mul_assoc_e1(y: GF4, z: GF4) {
    gf4_mul(GF4.one, gf4_mul(y, z)) = gf4_mul(gf4_mul(GF4.one, y), z)
} by {
    gf4_cases(y)
    if y = GF4.zero {
        gf4_mul_assoc_e1e0(z)
        gf4_mul(GF4.one, gf4_mul(y, z)) = gf4_mul(gf4_mul(GF4.one, y), z)
    }
    if y = GF4.one {
        gf4_mul_assoc_e1e1(z)
        gf4_mul(GF4.one, gf4_mul(y, z)) = gf4_mul(gf4_mul(GF4.one, y), z)
    }
    if y = GF4.x {
        gf4_mul_assoc_e1e2(z)
        gf4_mul(GF4.one, gf4_mul(y, z)) = gf4_mul(gf4_mul(GF4.one, y), z)
    }
    if y = GF4.x_plus_one {
        gf4_mul_assoc_e1e3(z)
        gf4_mul(GF4.one, gf4_mul(y, z)) = gf4_mul(gf4_mul(GF4.one, y), z)
    }
    gf4_mul(GF4.one, gf4_mul(y, z)) = gf4_mul(gf4_mul(GF4.one, y), z)
}

/// gf4_mul_assoc with the first argument fixed at GF4.x.
theorem gf4_mul_assoc_e2(y: GF4, z: GF4) {
    gf4_mul(GF4.x, gf4_mul(y, z)) = gf4_mul(gf4_mul(GF4.x, y), z)
} by {
    gf4_cases(y)
    if y = GF4.zero {
        gf4_mul_assoc_e2e0(z)
        gf4_mul(GF4.x, gf4_mul(y, z)) = gf4_mul(gf4_mul(GF4.x, y), z)
    }
    if y = GF4.one {
        gf4_mul_assoc_e2e1(z)
        gf4_mul(GF4.x, gf4_mul(y, z)) = gf4_mul(gf4_mul(GF4.x, y), z)
    }
    if y = GF4.x {
        gf4_mul_assoc_e2e2(z)
        gf4_mul(GF4.x, gf4_mul(y, z)) = gf4_mul(gf4_mul(GF4.x, y), z)
    }
    if y = GF4.x_plus_one {
        gf4_mul_assoc_e2e3(z)
        gf4_mul(GF4.x, gf4_mul(y, z)) = gf4_mul(gf4_mul(GF4.x, y), z)
    }
    gf4_mul(GF4.x, gf4_mul(y, z)) = gf4_mul(gf4_mul(GF4.x, y), z)
}

/// gf4_mul_assoc with the first argument fixed at GF4.x_plus_one.
theorem gf4_mul_assoc_e3(y: GF4, z: GF4) {
    gf4_mul(GF4.x_plus_one, gf4_mul(y, z)) = gf4_mul(gf4_mul(GF4.x_plus_one, y), z)
} by {
    gf4_cases(y)
    if y = GF4.zero {
        gf4_mul_assoc_e3e0(z)
        gf4_mul(GF4.x_plus_one, gf4_mul(y, z)) = gf4_mul(gf4_mul(GF4.x_plus_one, y), z)
    }
    if y = GF4.one {
        gf4_mul_assoc_e3e1(z)
        gf4_mul(GF4.x_plus_one, gf4_mul(y, z)) = gf4_mul(gf4_mul(GF4.x_plus_one, y), z)
    }
    if y = GF4.x {
        gf4_mul_assoc_e3e2(z)
        gf4_mul(GF4.x_plus_one, gf4_mul(y, z)) = gf4_mul(gf4_mul(GF4.x_plus_one, y), z)
    }
    if y = GF4.x_plus_one {
        gf4_mul_assoc_e3e3(z)
        gf4_mul(GF4.x_plus_one, gf4_mul(y, z)) = gf4_mul(gf4_mul(GF4.x_plus_one, y), z)
    }
    gf4_mul(GF4.x_plus_one, gf4_mul(y, z)) = gf4_mul(gf4_mul(GF4.x_plus_one, y), z)
}

/// gf4_mul_assoc.
theorem gf4_mul_assoc(x: GF4, y: GF4, z: GF4) {
    Mul.mul(x, Mul.mul(y, z)) = Mul.mul(Mul.mul(x, y), z)
} by {
    gf4_cases(x)
    if x = GF4.zero {
        gf4_mul_assoc_e0(y, z)
        gf4_mul(GF4.zero, gf4_mul(y, z)) = gf4_mul(gf4_mul(GF4.zero, y), z)
        gf4_mul(x, gf4_mul(y, z)) = gf4_mul(gf4_mul(x, y), z)
    }
    if x = GF4.one {
        gf4_mul_assoc_e1(y, z)
        gf4_mul(GF4.one, gf4_mul(y, z)) = gf4_mul(gf4_mul(GF4.one, y), z)
        gf4_mul(x, gf4_mul(y, z)) = gf4_mul(gf4_mul(x, y), z)
    }
    if x = GF4.x {
        gf4_mul_assoc_e2(y, z)
        gf4_mul(GF4.x, gf4_mul(y, z)) = gf4_mul(gf4_mul(GF4.x, y), z)
        gf4_mul(x, gf4_mul(y, z)) = gf4_mul(gf4_mul(x, y), z)
    }
    if x = GF4.x_plus_one {
        gf4_mul_assoc_e3(y, z)
        gf4_mul(GF4.x_plus_one, gf4_mul(y, z)) = gf4_mul(gf4_mul(GF4.x_plus_one, y), z)
        gf4_mul(x, gf4_mul(y, z)) = gf4_mul(gf4_mul(x, y), z)
    }
    gf4_mul(x, gf4_mul(y, z)) = gf4_mul(gf4_mul(x, y), z)
    Mul.mul(x, Mul.mul(y, z)) = Mul.mul(Mul.mul(x, y), z)
}

/// gf4_mul_add_left with the first two arguments fixed.
theorem gf4_mul_add_left_e0e0(z: GF4) {
    gf4_mul(GF4.zero, gf4_add(GF4.zero, z)) = gf4_add(gf4_mul(GF4.zero, GF4.zero), gf4_mul(GF4.zero, z))
} by {
    define w(a: GF4) -> Bool {
        gf4_mul(GF4.zero, gf4_add(GF4.zero, a)) = gf4_add(gf4_mul(GF4.zero, GF4.zero), gf4_mul(GF4.zero, a))
    }
    gf4_mul(GF4.zero, gf4_add(GF4.zero, GF4.zero)) = gf4_add(gf4_mul(GF4.zero, GF4.zero), gf4_mul(GF4.zero, GF4.zero))
    gf4_mul(GF4.zero, gf4_add(GF4.zero, GF4.one)) = gf4_add(gf4_mul(GF4.zero, GF4.zero), gf4_mul(GF4.zero, GF4.one))
    gf4_mul(GF4.zero, gf4_add(GF4.zero, GF4.x)) = gf4_add(gf4_mul(GF4.zero, GF4.zero), gf4_mul(GF4.zero, GF4.x))
    gf4_mul(GF4.zero, gf4_add(GF4.zero, GF4.x_plus_one)) = gf4_add(gf4_mul(GF4.zero, GF4.zero), gf4_mul(GF4.zero, GF4.x_plus_one))
    w(GF4.zero)
    w(GF4.one)
    w(GF4.x)
    w(GF4.x_plus_one)
    GF4.induction(w)
    w(z)
    gf4_mul(GF4.zero, gf4_add(GF4.zero, z)) = gf4_add(gf4_mul(GF4.zero, GF4.zero), gf4_mul(GF4.zero, z))
}

/// gf4_mul_add_left with the first two arguments fixed.
theorem gf4_mul_add_left_e0e1(z: GF4) {
    gf4_mul(GF4.zero, gf4_add(GF4.one, z)) = gf4_add(gf4_mul(GF4.zero, GF4.one), gf4_mul(GF4.zero, z))
} by {
    define w(a: GF4) -> Bool {
        gf4_mul(GF4.zero, gf4_add(GF4.one, a)) = gf4_add(gf4_mul(GF4.zero, GF4.one), gf4_mul(GF4.zero, a))
    }
    gf4_mul(GF4.zero, gf4_add(GF4.one, GF4.zero)) = gf4_add(gf4_mul(GF4.zero, GF4.one), gf4_mul(GF4.zero, GF4.zero))
    gf4_mul(GF4.zero, gf4_add(GF4.one, GF4.one)) = gf4_add(gf4_mul(GF4.zero, GF4.one), gf4_mul(GF4.zero, GF4.one))
    gf4_mul(GF4.zero, gf4_add(GF4.one, GF4.x)) = gf4_add(gf4_mul(GF4.zero, GF4.one), gf4_mul(GF4.zero, GF4.x))
    gf4_mul(GF4.zero, gf4_add(GF4.one, GF4.x_plus_one)) = gf4_add(gf4_mul(GF4.zero, GF4.one), gf4_mul(GF4.zero, GF4.x_plus_one))
    w(GF4.zero)
    w(GF4.one)
    w(GF4.x)
    w(GF4.x_plus_one)
    GF4.induction(w)
    w(z)
    gf4_mul(GF4.zero, gf4_add(GF4.one, z)) = gf4_add(gf4_mul(GF4.zero, GF4.one), gf4_mul(GF4.zero, z))
}

/// gf4_mul_add_left with the first two arguments fixed.
theorem gf4_mul_add_left_e0e2(z: GF4) {
    gf4_mul(GF4.zero, gf4_add(GF4.x, z)) = gf4_add(gf4_mul(GF4.zero, GF4.x), gf4_mul(GF4.zero, z))
} by {
    define w(a: GF4) -> Bool {
        gf4_mul(GF4.zero, gf4_add(GF4.x, a)) = gf4_add(gf4_mul(GF4.zero, GF4.x), gf4_mul(GF4.zero, a))
    }
    gf4_mul(GF4.zero, gf4_add(GF4.x, GF4.zero)) = gf4_add(gf4_mul(GF4.zero, GF4.x), gf4_mul(GF4.zero, GF4.zero))
    gf4_mul(GF4.zero, gf4_add(GF4.x, GF4.one)) = gf4_add(gf4_mul(GF4.zero, GF4.x), gf4_mul(GF4.zero, GF4.one))
    gf4_mul(GF4.zero, gf4_add(GF4.x, GF4.x)) = gf4_add(gf4_mul(GF4.zero, GF4.x), gf4_mul(GF4.zero, GF4.x))
    gf4_mul(GF4.zero, gf4_add(GF4.x, GF4.x_plus_one)) = gf4_add(gf4_mul(GF4.zero, GF4.x), gf4_mul(GF4.zero, GF4.x_plus_one))
    w(GF4.zero)
    w(GF4.one)
    w(GF4.x)
    w(GF4.x_plus_one)
    GF4.induction(w)
    w(z)
    gf4_mul(GF4.zero, gf4_add(GF4.x, z)) = gf4_add(gf4_mul(GF4.zero, GF4.x), gf4_mul(GF4.zero, z))
}

/// gf4_mul_add_left with the first two arguments fixed.
theorem gf4_mul_add_left_e0e3(z: GF4) {
    gf4_mul(GF4.zero, gf4_add(GF4.x_plus_one, z)) = gf4_add(gf4_mul(GF4.zero, GF4.x_plus_one), gf4_mul(GF4.zero, z))
} by {
    define w(a: GF4) -> Bool {
        gf4_mul(GF4.zero, gf4_add(GF4.x_plus_one, a)) = gf4_add(gf4_mul(GF4.zero, GF4.x_plus_one), gf4_mul(GF4.zero, a))
    }
    gf4_mul(GF4.zero, gf4_add(GF4.x_plus_one, GF4.zero)) = gf4_add(gf4_mul(GF4.zero, GF4.x_plus_one), gf4_mul(GF4.zero, GF4.zero))
    gf4_mul(GF4.zero, gf4_add(GF4.x_plus_one, GF4.one)) = gf4_add(gf4_mul(GF4.zero, GF4.x_plus_one), gf4_mul(GF4.zero, GF4.one))
    gf4_mul(GF4.zero, gf4_add(GF4.x_plus_one, GF4.x)) = gf4_add(gf4_mul(GF4.zero, GF4.x_plus_one), gf4_mul(GF4.zero, GF4.x))
    gf4_mul(GF4.zero, gf4_add(GF4.x_plus_one, GF4.x_plus_one)) = gf4_add(gf4_mul(GF4.zero, GF4.x_plus_one), gf4_mul(GF4.zero, GF4.x_plus_one))
    w(GF4.zero)
    w(GF4.one)
    w(GF4.x)
    w(GF4.x_plus_one)
    GF4.induction(w)
    w(z)
    gf4_mul(GF4.zero, gf4_add(GF4.x_plus_one, z)) = gf4_add(gf4_mul(GF4.zero, GF4.x_plus_one), gf4_mul(GF4.zero, z))
}

/// gf4_mul_add_left with the first two arguments fixed.
theorem gf4_mul_add_left_e1e0(z: GF4) {
    gf4_mul(GF4.one, gf4_add(GF4.zero, z)) = gf4_add(gf4_mul(GF4.one, GF4.zero), gf4_mul(GF4.one, z))
} by {
    define w(a: GF4) -> Bool {
        gf4_mul(GF4.one, gf4_add(GF4.zero, a)) = gf4_add(gf4_mul(GF4.one, GF4.zero), gf4_mul(GF4.one, a))
    }
    gf4_mul(GF4.one, gf4_add(GF4.zero, GF4.zero)) = gf4_add(gf4_mul(GF4.one, GF4.zero), gf4_mul(GF4.one, GF4.zero))
    gf4_mul(GF4.one, gf4_add(GF4.zero, GF4.one)) = gf4_add(gf4_mul(GF4.one, GF4.zero), gf4_mul(GF4.one, GF4.one))
    gf4_mul(GF4.one, gf4_add(GF4.zero, GF4.x)) = gf4_add(gf4_mul(GF4.one, GF4.zero), gf4_mul(GF4.one, GF4.x))
    gf4_mul(GF4.one, gf4_add(GF4.zero, GF4.x_plus_one)) = gf4_add(gf4_mul(GF4.one, GF4.zero), gf4_mul(GF4.one, GF4.x_plus_one))
    w(GF4.zero)
    w(GF4.one)
    w(GF4.x)
    w(GF4.x_plus_one)
    GF4.induction(w)
    w(z)
    gf4_mul(GF4.one, gf4_add(GF4.zero, z)) = gf4_add(gf4_mul(GF4.one, GF4.zero), gf4_mul(GF4.one, z))
}

/// gf4_mul_add_left with the first two arguments fixed.
theorem gf4_mul_add_left_e1e1(z: GF4) {
    gf4_mul(GF4.one, gf4_add(GF4.one, z)) = gf4_add(gf4_mul(GF4.one, GF4.one), gf4_mul(GF4.one, z))
} by {
    define w(a: GF4) -> Bool {
        gf4_mul(GF4.one, gf4_add(GF4.one, a)) = gf4_add(gf4_mul(GF4.one, GF4.one), gf4_mul(GF4.one, a))
    }
    gf4_mul(GF4.one, gf4_add(GF4.one, GF4.zero)) = gf4_add(gf4_mul(GF4.one, GF4.one), gf4_mul(GF4.one, GF4.zero))
    gf4_mul(GF4.one, gf4_add(GF4.one, GF4.one)) = gf4_add(gf4_mul(GF4.one, GF4.one), gf4_mul(GF4.one, GF4.one))
    gf4_mul(GF4.one, gf4_add(GF4.one, GF4.x)) = gf4_add(gf4_mul(GF4.one, GF4.one), gf4_mul(GF4.one, GF4.x))
    gf4_mul(GF4.one, gf4_add(GF4.one, GF4.x_plus_one)) = gf4_add(gf4_mul(GF4.one, GF4.one), gf4_mul(GF4.one, GF4.x_plus_one))
    w(GF4.zero)
    w(GF4.one)
    w(GF4.x)
    w(GF4.x_plus_one)
    GF4.induction(w)
    w(z)
    gf4_mul(GF4.one, gf4_add(GF4.one, z)) = gf4_add(gf4_mul(GF4.one, GF4.one), gf4_mul(GF4.one, z))
}

/// gf4_mul_add_left with the first two arguments fixed.
theorem gf4_mul_add_left_e1e2(z: GF4) {
    gf4_mul(GF4.one, gf4_add(GF4.x, z)) = gf4_add(gf4_mul(GF4.one, GF4.x), gf4_mul(GF4.one, z))
} by {
    define w(a: GF4) -> Bool {
        gf4_mul(GF4.one, gf4_add(GF4.x, a)) = gf4_add(gf4_mul(GF4.one, GF4.x), gf4_mul(GF4.one, a))
    }
    gf4_mul(GF4.one, gf4_add(GF4.x, GF4.zero)) = gf4_add(gf4_mul(GF4.one, GF4.x), gf4_mul(GF4.one, GF4.zero))
    gf4_mul(GF4.one, gf4_add(GF4.x, GF4.one)) = gf4_add(gf4_mul(GF4.one, GF4.x), gf4_mul(GF4.one, GF4.one))
    gf4_mul(GF4.one, gf4_add(GF4.x, GF4.x)) = gf4_add(gf4_mul(GF4.one, GF4.x), gf4_mul(GF4.one, GF4.x))
    gf4_mul(GF4.one, gf4_add(GF4.x, GF4.x_plus_one)) = gf4_add(gf4_mul(GF4.one, GF4.x), gf4_mul(GF4.one, GF4.x_plus_one))
    w(GF4.zero)
    w(GF4.one)
    w(GF4.x)
    w(GF4.x_plus_one)
    GF4.induction(w)
    w(z)
    gf4_mul(GF4.one, gf4_add(GF4.x, z)) = gf4_add(gf4_mul(GF4.one, GF4.x), gf4_mul(GF4.one, z))
}

/// gf4_mul_add_left with the first two arguments fixed.
theorem gf4_mul_add_left_e1e3(z: GF4) {
    gf4_mul(GF4.one, gf4_add(GF4.x_plus_one, z)) = gf4_add(gf4_mul(GF4.one, GF4.x_plus_one), gf4_mul(GF4.one, z))
} by {
    define w(a: GF4) -> Bool {
        gf4_mul(GF4.one, gf4_add(GF4.x_plus_one, a)) = gf4_add(gf4_mul(GF4.one, GF4.x_plus_one), gf4_mul(GF4.one, a))
    }
    gf4_mul(GF4.one, gf4_add(GF4.x_plus_one, GF4.zero)) = gf4_add(gf4_mul(GF4.one, GF4.x_plus_one), gf4_mul(GF4.one, GF4.zero))
    gf4_mul(GF4.one, gf4_add(GF4.x_plus_one, GF4.one)) = gf4_add(gf4_mul(GF4.one, GF4.x_plus_one), gf4_mul(GF4.one, GF4.one))
    gf4_mul(GF4.one, gf4_add(GF4.x_plus_one, GF4.x)) = gf4_add(gf4_mul(GF4.one, GF4.x_plus_one), gf4_mul(GF4.one, GF4.x))
    gf4_mul(GF4.one, gf4_add(GF4.x_plus_one, GF4.x_plus_one)) = gf4_add(gf4_mul(GF4.one, GF4.x_plus_one), gf4_mul(GF4.one, GF4.x_plus_one))
    w(GF4.zero)
    w(GF4.one)
    w(GF4.x)
    w(GF4.x_plus_one)
    GF4.induction(w)
    w(z)
    gf4_mul(GF4.one, gf4_add(GF4.x_plus_one, z)) = gf4_add(gf4_mul(GF4.one, GF4.x_plus_one), gf4_mul(GF4.one, z))
}

/// gf4_mul_add_left with the first two arguments fixed.
theorem gf4_mul_add_left_e2e0(z: GF4) {
    gf4_mul(GF4.x, gf4_add(GF4.zero, z)) = gf4_add(gf4_mul(GF4.x, GF4.zero), gf4_mul(GF4.x, z))
} by {
    define w(a: GF4) -> Bool {
        gf4_mul(GF4.x, gf4_add(GF4.zero, a)) = gf4_add(gf4_mul(GF4.x, GF4.zero), gf4_mul(GF4.x, a))
    }
    gf4_mul(GF4.x, gf4_add(GF4.zero, GF4.zero)) = gf4_add(gf4_mul(GF4.x, GF4.zero), gf4_mul(GF4.x, GF4.zero))
    gf4_mul(GF4.x, gf4_add(GF4.zero, GF4.one)) = gf4_add(gf4_mul(GF4.x, GF4.zero), gf4_mul(GF4.x, GF4.one))
    gf4_mul(GF4.x, gf4_add(GF4.zero, GF4.x)) = gf4_add(gf4_mul(GF4.x, GF4.zero), gf4_mul(GF4.x, GF4.x))
    gf4_mul(GF4.x, gf4_add(GF4.zero, GF4.x_plus_one)) = gf4_add(gf4_mul(GF4.x, GF4.zero), gf4_mul(GF4.x, GF4.x_plus_one))
    w(GF4.zero)
    w(GF4.one)
    w(GF4.x)
    w(GF4.x_plus_one)
    GF4.induction(w)
    w(z)
    gf4_mul(GF4.x, gf4_add(GF4.zero, z)) = gf4_add(gf4_mul(GF4.x, GF4.zero), gf4_mul(GF4.x, z))
}

/// gf4_mul_add_left with the first two arguments fixed.
theorem gf4_mul_add_left_e2e1(z: GF4) {
    gf4_mul(GF4.x, gf4_add(GF4.one, z)) = gf4_add(gf4_mul(GF4.x, GF4.one), gf4_mul(GF4.x, z))
} by {
    define w(a: GF4) -> Bool {
        gf4_mul(GF4.x, gf4_add(GF4.one, a)) = gf4_add(gf4_mul(GF4.x, GF4.one), gf4_mul(GF4.x, a))
    }
    gf4_mul(GF4.x, gf4_add(GF4.one, GF4.zero)) = gf4_add(gf4_mul(GF4.x, GF4.one), gf4_mul(GF4.x, GF4.zero))
    gf4_mul(GF4.x, gf4_add(GF4.one, GF4.one)) = gf4_add(gf4_mul(GF4.x, GF4.one), gf4_mul(GF4.x, GF4.one))
    gf4_mul(GF4.x, gf4_add(GF4.one, GF4.x)) = gf4_add(gf4_mul(GF4.x, GF4.one), gf4_mul(GF4.x, GF4.x))
    gf4_mul(GF4.x, gf4_add(GF4.one, GF4.x_plus_one)) = gf4_add(gf4_mul(GF4.x, GF4.one), gf4_mul(GF4.x, GF4.x_plus_one))
    w(GF4.zero)
    w(GF4.one)
    w(GF4.x)
    w(GF4.x_plus_one)
    GF4.induction(w)
    w(z)
    gf4_mul(GF4.x, gf4_add(GF4.one, z)) = gf4_add(gf4_mul(GF4.x, GF4.one), gf4_mul(GF4.x, z))
}

/// gf4_mul_add_left with the first two arguments fixed.
theorem gf4_mul_add_left_e2e2(z: GF4) {
    gf4_mul(GF4.x, gf4_add(GF4.x, z)) = gf4_add(gf4_mul(GF4.x, GF4.x), gf4_mul(GF4.x, z))
} by {
    define w(a: GF4) -> Bool {
        gf4_mul(GF4.x, gf4_add(GF4.x, a)) = gf4_add(gf4_mul(GF4.x, GF4.x), gf4_mul(GF4.x, a))
    }
    gf4_mul(GF4.x, gf4_add(GF4.x, GF4.zero)) = gf4_add(gf4_mul(GF4.x, GF4.x), gf4_mul(GF4.x, GF4.zero))
    gf4_mul(GF4.x, gf4_add(GF4.x, GF4.one)) = gf4_add(gf4_mul(GF4.x, GF4.x), gf4_mul(GF4.x, GF4.one))
    gf4_mul(GF4.x, gf4_add(GF4.x, GF4.x)) = gf4_add(gf4_mul(GF4.x, GF4.x), gf4_mul(GF4.x, GF4.x))
    gf4_mul(GF4.x, gf4_add(GF4.x, GF4.x_plus_one)) = gf4_add(gf4_mul(GF4.x, GF4.x), gf4_mul(GF4.x, GF4.x_plus_one))
    w(GF4.zero)
    w(GF4.one)
    w(GF4.x)
    w(GF4.x_plus_one)
    GF4.induction(w)
    w(z)
    gf4_mul(GF4.x, gf4_add(GF4.x, z)) = gf4_add(gf4_mul(GF4.x, GF4.x), gf4_mul(GF4.x, z))
}

/// gf4_mul_add_left with the first two arguments fixed.
theorem gf4_mul_add_left_e2e3(z: GF4) {
    gf4_mul(GF4.x, gf4_add(GF4.x_plus_one, z)) = gf4_add(gf4_mul(GF4.x, GF4.x_plus_one), gf4_mul(GF4.x, z))
} by {
    define w(a: GF4) -> Bool {
        gf4_mul(GF4.x, gf4_add(GF4.x_plus_one, a)) = gf4_add(gf4_mul(GF4.x, GF4.x_plus_one), gf4_mul(GF4.x, a))
    }
    gf4_mul(GF4.x, gf4_add(GF4.x_plus_one, GF4.zero)) = gf4_add(gf4_mul(GF4.x, GF4.x_plus_one), gf4_mul(GF4.x, GF4.zero))
    gf4_mul(GF4.x, gf4_add(GF4.x_plus_one, GF4.one)) = gf4_add(gf4_mul(GF4.x, GF4.x_plus_one), gf4_mul(GF4.x, GF4.one))
    gf4_mul(GF4.x, gf4_add(GF4.x_plus_one, GF4.x)) = gf4_add(gf4_mul(GF4.x, GF4.x_plus_one), gf4_mul(GF4.x, GF4.x))
    gf4_mul(GF4.x, gf4_add(GF4.x_plus_one, GF4.x_plus_one)) = gf4_add(gf4_mul(GF4.x, GF4.x_plus_one), gf4_mul(GF4.x, GF4.x_plus_one))
    w(GF4.zero)
    w(GF4.one)
    w(GF4.x)
    w(GF4.x_plus_one)
    GF4.induction(w)
    w(z)
    gf4_mul(GF4.x, gf4_add(GF4.x_plus_one, z)) = gf4_add(gf4_mul(GF4.x, GF4.x_plus_one), gf4_mul(GF4.x, z))
}

/// gf4_mul_add_left with the first two arguments fixed.
theorem gf4_mul_add_left_e3e0(z: GF4) {
    gf4_mul(GF4.x_plus_one, gf4_add(GF4.zero, z)) = gf4_add(gf4_mul(GF4.x_plus_one, GF4.zero), gf4_mul(GF4.x_plus_one, z))
} by {
    define w(a: GF4) -> Bool {
        gf4_mul(GF4.x_plus_one, gf4_add(GF4.zero, a)) = gf4_add(gf4_mul(GF4.x_plus_one, GF4.zero), gf4_mul(GF4.x_plus_one, a))
    }
    gf4_mul(GF4.x_plus_one, gf4_add(GF4.zero, GF4.zero)) = gf4_add(gf4_mul(GF4.x_plus_one, GF4.zero), gf4_mul(GF4.x_plus_one, GF4.zero))
    gf4_mul(GF4.x_plus_one, gf4_add(GF4.zero, GF4.one)) = gf4_add(gf4_mul(GF4.x_plus_one, GF4.zero), gf4_mul(GF4.x_plus_one, GF4.one))
    gf4_mul(GF4.x_plus_one, gf4_add(GF4.zero, GF4.x)) = gf4_add(gf4_mul(GF4.x_plus_one, GF4.zero), gf4_mul(GF4.x_plus_one, GF4.x))
    gf4_mul(GF4.x_plus_one, gf4_add(GF4.zero, GF4.x_plus_one)) = gf4_add(gf4_mul(GF4.x_plus_one, GF4.zero), gf4_mul(GF4.x_plus_one, GF4.x_plus_one))
    w(GF4.zero)
    w(GF4.one)
    w(GF4.x)
    w(GF4.x_plus_one)
    GF4.induction(w)
    w(z)
    gf4_mul(GF4.x_plus_one, gf4_add(GF4.zero, z)) = gf4_add(gf4_mul(GF4.x_plus_one, GF4.zero), gf4_mul(GF4.x_plus_one, z))
}

/// gf4_mul_add_left with the first two arguments fixed.
theorem gf4_mul_add_left_e3e1(z: GF4) {
    gf4_mul(GF4.x_plus_one, gf4_add(GF4.one, z)) = gf4_add(gf4_mul(GF4.x_plus_one, GF4.one), gf4_mul(GF4.x_plus_one, z))
} by {
    define w(a: GF4) -> Bool {
        gf4_mul(GF4.x_plus_one, gf4_add(GF4.one, a)) = gf4_add(gf4_mul(GF4.x_plus_one, GF4.one), gf4_mul(GF4.x_plus_one, a))
    }
    gf4_mul(GF4.x_plus_one, gf4_add(GF4.one, GF4.zero)) = gf4_add(gf4_mul(GF4.x_plus_one, GF4.one), gf4_mul(GF4.x_plus_one, GF4.zero))
    gf4_mul(GF4.x_plus_one, gf4_add(GF4.one, GF4.one)) = gf4_add(gf4_mul(GF4.x_plus_one, GF4.one), gf4_mul(GF4.x_plus_one, GF4.one))
    gf4_mul(GF4.x_plus_one, gf4_add(GF4.one, GF4.x)) = gf4_add(gf4_mul(GF4.x_plus_one, GF4.one), gf4_mul(GF4.x_plus_one, GF4.x))
    gf4_mul(GF4.x_plus_one, gf4_add(GF4.one, GF4.x_plus_one)) = gf4_add(gf4_mul(GF4.x_plus_one, GF4.one), gf4_mul(GF4.x_plus_one, GF4.x_plus_one))
    w(GF4.zero)
    w(GF4.one)
    w(GF4.x)
    w(GF4.x_plus_one)
    GF4.induction(w)
    w(z)
    gf4_mul(GF4.x_plus_one, gf4_add(GF4.one, z)) = gf4_add(gf4_mul(GF4.x_plus_one, GF4.one), gf4_mul(GF4.x_plus_one, z))
}

/// gf4_mul_add_left with the first two arguments fixed.
theorem gf4_mul_add_left_e3e2(z: GF4) {
    gf4_mul(GF4.x_plus_one, gf4_add(GF4.x, z)) = gf4_add(gf4_mul(GF4.x_plus_one, GF4.x), gf4_mul(GF4.x_plus_one, z))
} by {
    define w(a: GF4) -> Bool {
        gf4_mul(GF4.x_plus_one, gf4_add(GF4.x, a)) = gf4_add(gf4_mul(GF4.x_plus_one, GF4.x), gf4_mul(GF4.x_plus_one, a))
    }
    gf4_mul(GF4.x_plus_one, gf4_add(GF4.x, GF4.zero)) = gf4_add(gf4_mul(GF4.x_plus_one, GF4.x), gf4_mul(GF4.x_plus_one, GF4.zero))
    gf4_mul(GF4.x_plus_one, gf4_add(GF4.x, GF4.one)) = gf4_add(gf4_mul(GF4.x_plus_one, GF4.x), gf4_mul(GF4.x_plus_one, GF4.one))
    gf4_mul(GF4.x_plus_one, gf4_add(GF4.x, GF4.x)) = gf4_add(gf4_mul(GF4.x_plus_one, GF4.x), gf4_mul(GF4.x_plus_one, GF4.x))
    gf4_mul(GF4.x_plus_one, gf4_add(GF4.x, GF4.x_plus_one)) = gf4_add(gf4_mul(GF4.x_plus_one, GF4.x), gf4_mul(GF4.x_plus_one, GF4.x_plus_one))
    w(GF4.zero)
    w(GF4.one)
    w(GF4.x)
    w(GF4.x_plus_one)
    GF4.induction(w)
    w(z)
    gf4_mul(GF4.x_plus_one, gf4_add(GF4.x, z)) = gf4_add(gf4_mul(GF4.x_plus_one, GF4.x), gf4_mul(GF4.x_plus_one, z))
}

/// gf4_mul_add_left with the first two arguments fixed.
theorem gf4_mul_add_left_e3e3(z: GF4) {
    gf4_mul(GF4.x_plus_one, gf4_add(GF4.x_plus_one, z)) = gf4_add(gf4_mul(GF4.x_plus_one, GF4.x_plus_one), gf4_mul(GF4.x_plus_one, z))
} by {
    define w(a: GF4) -> Bool {
        gf4_mul(GF4.x_plus_one, gf4_add(GF4.x_plus_one, a)) = gf4_add(gf4_mul(GF4.x_plus_one, GF4.x_plus_one), gf4_mul(GF4.x_plus_one, a))
    }
    gf4_mul(GF4.x_plus_one, gf4_add(GF4.x_plus_one, GF4.zero)) = gf4_add(gf4_mul(GF4.x_plus_one, GF4.x_plus_one), gf4_mul(GF4.x_plus_one, GF4.zero))
    gf4_mul(GF4.x_plus_one, gf4_add(GF4.x_plus_one, GF4.one)) = gf4_add(gf4_mul(GF4.x_plus_one, GF4.x_plus_one), gf4_mul(GF4.x_plus_one, GF4.one))
    gf4_mul(GF4.x_plus_one, gf4_add(GF4.x_plus_one, GF4.x)) = gf4_add(gf4_mul(GF4.x_plus_one, GF4.x_plus_one), gf4_mul(GF4.x_plus_one, GF4.x))
    gf4_mul(GF4.x_plus_one, gf4_add(GF4.x_plus_one, GF4.x_plus_one)) = gf4_add(gf4_mul(GF4.x_plus_one, GF4.x_plus_one), gf4_mul(GF4.x_plus_one, GF4.x_plus_one))
    w(GF4.zero)
    w(GF4.one)
    w(GF4.x)
    w(GF4.x_plus_one)
    GF4.induction(w)
    w(z)
    gf4_mul(GF4.x_plus_one, gf4_add(GF4.x_plus_one, z)) = gf4_add(gf4_mul(GF4.x_plus_one, GF4.x_plus_one), gf4_mul(GF4.x_plus_one, z))
}

/// gf4_mul_add_left with the first argument fixed at GF4.zero.
theorem gf4_mul_add_left_e0(y: GF4, z: GF4) {
    gf4_mul(GF4.zero, gf4_add(y, z)) = gf4_add(gf4_mul(GF4.zero, y), gf4_mul(GF4.zero, z))
} by {
    gf4_cases(y)
    if y = GF4.zero {
        gf4_mul_add_left_e0e0(z)
        gf4_mul(GF4.zero, gf4_add(y, z)) = gf4_add(gf4_mul(GF4.zero, y), gf4_mul(GF4.zero, z))
    }
    if y = GF4.one {
        gf4_mul_add_left_e0e1(z)
        gf4_mul(GF4.zero, gf4_add(y, z)) = gf4_add(gf4_mul(GF4.zero, y), gf4_mul(GF4.zero, z))
    }
    if y = GF4.x {
        gf4_mul_add_left_e0e2(z)
        gf4_mul(GF4.zero, gf4_add(y, z)) = gf4_add(gf4_mul(GF4.zero, y), gf4_mul(GF4.zero, z))
    }
    if y = GF4.x_plus_one {
        gf4_mul_add_left_e0e3(z)
        gf4_mul(GF4.zero, gf4_add(y, z)) = gf4_add(gf4_mul(GF4.zero, y), gf4_mul(GF4.zero, z))
    }
    gf4_mul(GF4.zero, gf4_add(y, z)) = gf4_add(gf4_mul(GF4.zero, y), gf4_mul(GF4.zero, z))
}

/// gf4_mul_add_left with the first argument fixed at GF4.one.
theorem gf4_mul_add_left_e1(y: GF4, z: GF4) {
    gf4_mul(GF4.one, gf4_add(y, z)) = gf4_add(gf4_mul(GF4.one, y), gf4_mul(GF4.one, z))
} by {
    gf4_cases(y)
    if y = GF4.zero {
        gf4_mul_add_left_e1e0(z)
        gf4_mul(GF4.one, gf4_add(y, z)) = gf4_add(gf4_mul(GF4.one, y), gf4_mul(GF4.one, z))
    }
    if y = GF4.one {
        gf4_mul_add_left_e1e1(z)
        gf4_mul(GF4.one, gf4_add(y, z)) = gf4_add(gf4_mul(GF4.one, y), gf4_mul(GF4.one, z))
    }
    if y = GF4.x {
        gf4_mul_add_left_e1e2(z)
        gf4_mul(GF4.one, gf4_add(y, z)) = gf4_add(gf4_mul(GF4.one, y), gf4_mul(GF4.one, z))
    }
    if y = GF4.x_plus_one {
        gf4_mul_add_left_e1e3(z)
        gf4_mul(GF4.one, gf4_add(y, z)) = gf4_add(gf4_mul(GF4.one, y), gf4_mul(GF4.one, z))
    }
    gf4_mul(GF4.one, gf4_add(y, z)) = gf4_add(gf4_mul(GF4.one, y), gf4_mul(GF4.one, z))
}

/// gf4_mul_add_left with the first argument fixed at GF4.x.
theorem gf4_mul_add_left_e2(y: GF4, z: GF4) {
    gf4_mul(GF4.x, gf4_add(y, z)) = gf4_add(gf4_mul(GF4.x, y), gf4_mul(GF4.x, z))
} by {
    gf4_cases(y)
    if y = GF4.zero {
        gf4_mul_add_left_e2e0(z)
        gf4_mul(GF4.x, gf4_add(y, z)) = gf4_add(gf4_mul(GF4.x, y), gf4_mul(GF4.x, z))
    }
    if y = GF4.one {
        gf4_mul_add_left_e2e1(z)
        gf4_mul(GF4.x, gf4_add(y, z)) = gf4_add(gf4_mul(GF4.x, y), gf4_mul(GF4.x, z))
    }
    if y = GF4.x {
        gf4_mul_add_left_e2e2(z)
        gf4_mul(GF4.x, gf4_add(y, z)) = gf4_add(gf4_mul(GF4.x, y), gf4_mul(GF4.x, z))
    }
    if y = GF4.x_plus_one {
        gf4_mul_add_left_e2e3(z)
        gf4_mul(GF4.x, gf4_add(y, z)) = gf4_add(gf4_mul(GF4.x, y), gf4_mul(GF4.x, z))
    }
    gf4_mul(GF4.x, gf4_add(y, z)) = gf4_add(gf4_mul(GF4.x, y), gf4_mul(GF4.x, z))
}

/// gf4_mul_add_left with the first argument fixed at GF4.x_plus_one.
theorem gf4_mul_add_left_e3(y: GF4, z: GF4) {
    gf4_mul(GF4.x_plus_one, gf4_add(y, z)) = gf4_add(gf4_mul(GF4.x_plus_one, y), gf4_mul(GF4.x_plus_one, z))
} by {
    gf4_cases(y)
    if y = GF4.zero {
        gf4_mul_add_left_e3e0(z)
        gf4_mul(GF4.x_plus_one, gf4_add(y, z)) = gf4_add(gf4_mul(GF4.x_plus_one, y), gf4_mul(GF4.x_plus_one, z))
    }
    if y = GF4.one {
        gf4_mul_add_left_e3e1(z)
        gf4_mul(GF4.x_plus_one, gf4_add(y, z)) = gf4_add(gf4_mul(GF4.x_plus_one, y), gf4_mul(GF4.x_plus_one, z))
    }
    if y = GF4.x {
        gf4_mul_add_left_e3e2(z)
        gf4_mul(GF4.x_plus_one, gf4_add(y, z)) = gf4_add(gf4_mul(GF4.x_plus_one, y), gf4_mul(GF4.x_plus_one, z))
    }
    if y = GF4.x_plus_one {
        gf4_mul_add_left_e3e3(z)
        gf4_mul(GF4.x_plus_one, gf4_add(y, z)) = gf4_add(gf4_mul(GF4.x_plus_one, y), gf4_mul(GF4.x_plus_one, z))
    }
    gf4_mul(GF4.x_plus_one, gf4_add(y, z)) = gf4_add(gf4_mul(GF4.x_plus_one, y), gf4_mul(GF4.x_plus_one, z))
}

/// gf4_mul_add_left.
theorem gf4_mul_add_left(x: GF4, y: GF4, z: GF4) {
    Mul.mul(x, Add.add(y, z)) = Add.add(Mul.mul(x, y), Mul.mul(x, z))
} by {
    gf4_cases(x)
    if x = GF4.zero {
        gf4_mul_add_left_e0(y, z)
        gf4_mul(GF4.zero, gf4_add(y, z)) = gf4_add(gf4_mul(GF4.zero, y), gf4_mul(GF4.zero, z))
        gf4_mul(x, gf4_add(y, z)) = gf4_add(gf4_mul(x, y), gf4_mul(x, z))
    }
    if x = GF4.one {
        gf4_mul_add_left_e1(y, z)
        gf4_mul(GF4.one, gf4_add(y, z)) = gf4_add(gf4_mul(GF4.one, y), gf4_mul(GF4.one, z))
        gf4_mul(x, gf4_add(y, z)) = gf4_add(gf4_mul(x, y), gf4_mul(x, z))
    }
    if x = GF4.x {
        gf4_mul_add_left_e2(y, z)
        gf4_mul(GF4.x, gf4_add(y, z)) = gf4_add(gf4_mul(GF4.x, y), gf4_mul(GF4.x, z))
        gf4_mul(x, gf4_add(y, z)) = gf4_add(gf4_mul(x, y), gf4_mul(x, z))
    }
    if x = GF4.x_plus_one {
        gf4_mul_add_left_e3(y, z)
        gf4_mul(GF4.x_plus_one, gf4_add(y, z)) = gf4_add(gf4_mul(GF4.x_plus_one, y), gf4_mul(GF4.x_plus_one, z))
        gf4_mul(x, gf4_add(y, z)) = gf4_add(gf4_mul(x, y), gf4_mul(x, z))
    }
    gf4_mul(x, gf4_add(y, z)) = gf4_add(gf4_mul(x, y), gf4_mul(x, z))
    Mul.mul(x, Add.add(y, z)) = Add.add(Mul.mul(x, y), Mul.mul(x, z))
}

/// gf4_mul_add_right with the first two arguments fixed.
theorem gf4_mul_add_right_e0e0(z: GF4) {
    gf4_mul(gf4_add(GF4.zero, GF4.zero), z) = gf4_add(gf4_mul(GF4.zero, z), gf4_mul(GF4.zero, z))
} by {
    define w(a: GF4) -> Bool {
        gf4_mul(gf4_add(GF4.zero, GF4.zero), a) = gf4_add(gf4_mul(GF4.zero, a), gf4_mul(GF4.zero, a))
    }
    gf4_mul(gf4_add(GF4.zero, GF4.zero), GF4.zero) = gf4_add(gf4_mul(GF4.zero, GF4.zero), gf4_mul(GF4.zero, GF4.zero))
    gf4_mul(gf4_add(GF4.zero, GF4.zero), GF4.one) = gf4_add(gf4_mul(GF4.zero, GF4.one), gf4_mul(GF4.zero, GF4.one))
    gf4_mul(gf4_add(GF4.zero, GF4.zero), GF4.x) = gf4_add(gf4_mul(GF4.zero, GF4.x), gf4_mul(GF4.zero, GF4.x))
    gf4_mul(gf4_add(GF4.zero, GF4.zero), GF4.x_plus_one) = gf4_add(gf4_mul(GF4.zero, GF4.x_plus_one), gf4_mul(GF4.zero, GF4.x_plus_one))
    w(GF4.zero)
    w(GF4.one)
    w(GF4.x)
    w(GF4.x_plus_one)
    GF4.induction(w)
    w(z)
    gf4_mul(gf4_add(GF4.zero, GF4.zero), z) = gf4_add(gf4_mul(GF4.zero, z), gf4_mul(GF4.zero, z))
}

/// gf4_mul_add_right with the first two arguments fixed.
theorem gf4_mul_add_right_e0e1(z: GF4) {
    gf4_mul(gf4_add(GF4.zero, GF4.one), z) = gf4_add(gf4_mul(GF4.zero, z), gf4_mul(GF4.one, z))
} by {
    define w(a: GF4) -> Bool {
        gf4_mul(gf4_add(GF4.zero, GF4.one), a) = gf4_add(gf4_mul(GF4.zero, a), gf4_mul(GF4.one, a))
    }
    gf4_mul(gf4_add(GF4.zero, GF4.one), GF4.zero) = gf4_add(gf4_mul(GF4.zero, GF4.zero), gf4_mul(GF4.one, GF4.zero))
    gf4_mul(gf4_add(GF4.zero, GF4.one), GF4.one) = gf4_add(gf4_mul(GF4.zero, GF4.one), gf4_mul(GF4.one, GF4.one))
    gf4_mul(gf4_add(GF4.zero, GF4.one), GF4.x) = gf4_add(gf4_mul(GF4.zero, GF4.x), gf4_mul(GF4.one, GF4.x))
    gf4_mul(gf4_add(GF4.zero, GF4.one), GF4.x_plus_one) = gf4_add(gf4_mul(GF4.zero, GF4.x_plus_one), gf4_mul(GF4.one, GF4.x_plus_one))
    w(GF4.zero)
    w(GF4.one)
    w(GF4.x)
    w(GF4.x_plus_one)
    GF4.induction(w)
    w(z)
    gf4_mul(gf4_add(GF4.zero, GF4.one), z) = gf4_add(gf4_mul(GF4.zero, z), gf4_mul(GF4.one, z))
}

/// gf4_mul_add_right with the first two arguments fixed.
theorem gf4_mul_add_right_e0e2(z: GF4) {
    gf4_mul(gf4_add(GF4.zero, GF4.x), z) = gf4_add(gf4_mul(GF4.zero, z), gf4_mul(GF4.x, z))
} by {
    define w(a: GF4) -> Bool {
        gf4_mul(gf4_add(GF4.zero, GF4.x), a) = gf4_add(gf4_mul(GF4.zero, a), gf4_mul(GF4.x, a))
    }
    gf4_mul(gf4_add(GF4.zero, GF4.x), GF4.zero) = gf4_add(gf4_mul(GF4.zero, GF4.zero), gf4_mul(GF4.x, GF4.zero))
    gf4_mul(gf4_add(GF4.zero, GF4.x), GF4.one) = gf4_add(gf4_mul(GF4.zero, GF4.one), gf4_mul(GF4.x, GF4.one))
    gf4_mul(gf4_add(GF4.zero, GF4.x), GF4.x) = gf4_add(gf4_mul(GF4.zero, GF4.x), gf4_mul(GF4.x, GF4.x))
    gf4_mul(gf4_add(GF4.zero, GF4.x), GF4.x_plus_one) = gf4_add(gf4_mul(GF4.zero, GF4.x_plus_one), gf4_mul(GF4.x, GF4.x_plus_one))
    w(GF4.zero)
    w(GF4.one)
    w(GF4.x)
    w(GF4.x_plus_one)
    GF4.induction(w)
    w(z)
    gf4_mul(gf4_add(GF4.zero, GF4.x), z) = gf4_add(gf4_mul(GF4.zero, z), gf4_mul(GF4.x, z))
}

/// gf4_mul_add_right with the first two arguments fixed.
theorem gf4_mul_add_right_e0e3(z: GF4) {
    gf4_mul(gf4_add(GF4.zero, GF4.x_plus_one), z) = gf4_add(gf4_mul(GF4.zero, z), gf4_mul(GF4.x_plus_one, z))
} by {
    define w(a: GF4) -> Bool {
        gf4_mul(gf4_add(GF4.zero, GF4.x_plus_one), a) = gf4_add(gf4_mul(GF4.zero, a), gf4_mul(GF4.x_plus_one, a))
    }
    gf4_mul(gf4_add(GF4.zero, GF4.x_plus_one), GF4.zero) = gf4_add(gf4_mul(GF4.zero, GF4.zero), gf4_mul(GF4.x_plus_one, GF4.zero))
    gf4_mul(gf4_add(GF4.zero, GF4.x_plus_one), GF4.one) = gf4_add(gf4_mul(GF4.zero, GF4.one), gf4_mul(GF4.x_plus_one, GF4.one))
    gf4_mul(gf4_add(GF4.zero, GF4.x_plus_one), GF4.x) = gf4_add(gf4_mul(GF4.zero, GF4.x), gf4_mul(GF4.x_plus_one, GF4.x))
    gf4_mul(gf4_add(GF4.zero, GF4.x_plus_one), GF4.x_plus_one) = gf4_add(gf4_mul(GF4.zero, GF4.x_plus_one), gf4_mul(GF4.x_plus_one, GF4.x_plus_one))
    w(GF4.zero)
    w(GF4.one)
    w(GF4.x)
    w(GF4.x_plus_one)
    GF4.induction(w)
    w(z)
    gf4_mul(gf4_add(GF4.zero, GF4.x_plus_one), z) = gf4_add(gf4_mul(GF4.zero, z), gf4_mul(GF4.x_plus_one, z))
}

/// gf4_mul_add_right with the first two arguments fixed.
theorem gf4_mul_add_right_e1e0(z: GF4) {
    gf4_mul(gf4_add(GF4.one, GF4.zero), z) = gf4_add(gf4_mul(GF4.one, z), gf4_mul(GF4.zero, z))
} by {
    define w(a: GF4) -> Bool {
        gf4_mul(gf4_add(GF4.one, GF4.zero), a) = gf4_add(gf4_mul(GF4.one, a), gf4_mul(GF4.zero, a))
    }
    gf4_mul(gf4_add(GF4.one, GF4.zero), GF4.zero) = gf4_add(gf4_mul(GF4.one, GF4.zero), gf4_mul(GF4.zero, GF4.zero))
    gf4_mul(gf4_add(GF4.one, GF4.zero), GF4.one) = gf4_add(gf4_mul(GF4.one, GF4.one), gf4_mul(GF4.zero, GF4.one))
    gf4_mul(gf4_add(GF4.one, GF4.zero), GF4.x) = gf4_add(gf4_mul(GF4.one, GF4.x), gf4_mul(GF4.zero, GF4.x))
    gf4_mul(gf4_add(GF4.one, GF4.zero), GF4.x_plus_one) = gf4_add(gf4_mul(GF4.one, GF4.x_plus_one), gf4_mul(GF4.zero, GF4.x_plus_one))
    w(GF4.zero)
    w(GF4.one)
    w(GF4.x)
    w(GF4.x_plus_one)
    GF4.induction(w)
    w(z)
    gf4_mul(gf4_add(GF4.one, GF4.zero), z) = gf4_add(gf4_mul(GF4.one, z), gf4_mul(GF4.zero, z))
}

/// gf4_mul_add_right with the first two arguments fixed.
theorem gf4_mul_add_right_e1e1(z: GF4) {
    gf4_mul(gf4_add(GF4.one, GF4.one), z) = gf4_add(gf4_mul(GF4.one, z), gf4_mul(GF4.one, z))
} by {
    define w(a: GF4) -> Bool {
        gf4_mul(gf4_add(GF4.one, GF4.one), a) = gf4_add(gf4_mul(GF4.one, a), gf4_mul(GF4.one, a))
    }
    gf4_mul(gf4_add(GF4.one, GF4.one), GF4.zero) = gf4_add(gf4_mul(GF4.one, GF4.zero), gf4_mul(GF4.one, GF4.zero))
    gf4_mul(gf4_add(GF4.one, GF4.one), GF4.one) = gf4_add(gf4_mul(GF4.one, GF4.one), gf4_mul(GF4.one, GF4.one))
    gf4_mul(gf4_add(GF4.one, GF4.one), GF4.x) = gf4_add(gf4_mul(GF4.one, GF4.x), gf4_mul(GF4.one, GF4.x))
    gf4_mul(gf4_add(GF4.one, GF4.one), GF4.x_plus_one) = gf4_add(gf4_mul(GF4.one, GF4.x_plus_one), gf4_mul(GF4.one, GF4.x_plus_one))
    w(GF4.zero)
    w(GF4.one)
    w(GF4.x)
    w(GF4.x_plus_one)
    GF4.induction(w)
    w(z)
    gf4_mul(gf4_add(GF4.one, GF4.one), z) = gf4_add(gf4_mul(GF4.one, z), gf4_mul(GF4.one, z))
}

/// gf4_mul_add_right with the first two arguments fixed.
theorem gf4_mul_add_right_e1e2(z: GF4) {
    gf4_mul(gf4_add(GF4.one, GF4.x), z) = gf4_add(gf4_mul(GF4.one, z), gf4_mul(GF4.x, z))
} by {
    define w(a: GF4) -> Bool {
        gf4_mul(gf4_add(GF4.one, GF4.x), a) = gf4_add(gf4_mul(GF4.one, a), gf4_mul(GF4.x, a))
    }
    gf4_mul(gf4_add(GF4.one, GF4.x), GF4.zero) = gf4_add(gf4_mul(GF4.one, GF4.zero), gf4_mul(GF4.x, GF4.zero))
    gf4_mul(gf4_add(GF4.one, GF4.x), GF4.one) = gf4_add(gf4_mul(GF4.one, GF4.one), gf4_mul(GF4.x, GF4.one))
    gf4_mul(gf4_add(GF4.one, GF4.x), GF4.x) = gf4_add(gf4_mul(GF4.one, GF4.x), gf4_mul(GF4.x, GF4.x))
    gf4_mul(gf4_add(GF4.one, GF4.x), GF4.x_plus_one) = gf4_add(gf4_mul(GF4.one, GF4.x_plus_one), gf4_mul(GF4.x, GF4.x_plus_one))
    w(GF4.zero)
    w(GF4.one)
    w(GF4.x)
    w(GF4.x_plus_one)
    GF4.induction(w)
    w(z)
    gf4_mul(gf4_add(GF4.one, GF4.x), z) = gf4_add(gf4_mul(GF4.one, z), gf4_mul(GF4.x, z))
}

/// gf4_mul_add_right with the first two arguments fixed.
theorem gf4_mul_add_right_e1e3(z: GF4) {
    gf4_mul(gf4_add(GF4.one, GF4.x_plus_one), z) = gf4_add(gf4_mul(GF4.one, z), gf4_mul(GF4.x_plus_one, z))
} by {
    define w(a: GF4) -> Bool {
        gf4_mul(gf4_add(GF4.one, GF4.x_plus_one), a) = gf4_add(gf4_mul(GF4.one, a), gf4_mul(GF4.x_plus_one, a))
    }
    gf4_mul(gf4_add(GF4.one, GF4.x_plus_one), GF4.zero) = gf4_add(gf4_mul(GF4.one, GF4.zero), gf4_mul(GF4.x_plus_one, GF4.zero))
    gf4_mul(gf4_add(GF4.one, GF4.x_plus_one), GF4.one) = gf4_add(gf4_mul(GF4.one, GF4.one), gf4_mul(GF4.x_plus_one, GF4.one))
    gf4_mul(gf4_add(GF4.one, GF4.x_plus_one), GF4.x) = gf4_add(gf4_mul(GF4.one, GF4.x), gf4_mul(GF4.x_plus_one, GF4.x))
    gf4_mul(gf4_add(GF4.one, GF4.x_plus_one), GF4.x_plus_one) = gf4_add(gf4_mul(GF4.one, GF4.x_plus_one), gf4_mul(GF4.x_plus_one, GF4.x_plus_one))
    w(GF4.zero)
    w(GF4.one)
    w(GF4.x)
    w(GF4.x_plus_one)
    GF4.induction(w)
    w(z)
    gf4_mul(gf4_add(GF4.one, GF4.x_plus_one), z) = gf4_add(gf4_mul(GF4.one, z), gf4_mul(GF4.x_plus_one, z))
}

/// gf4_mul_add_right with the first two arguments fixed.
theorem gf4_mul_add_right_e2e0(z: GF4) {
    gf4_mul(gf4_add(GF4.x, GF4.zero), z) = gf4_add(gf4_mul(GF4.x, z), gf4_mul(GF4.zero, z))
} by {
    define w(a: GF4) -> Bool {
        gf4_mul(gf4_add(GF4.x, GF4.zero), a) = gf4_add(gf4_mul(GF4.x, a), gf4_mul(GF4.zero, a))
    }
    gf4_mul(gf4_add(GF4.x, GF4.zero), GF4.zero) = gf4_add(gf4_mul(GF4.x, GF4.zero), gf4_mul(GF4.zero, GF4.zero))
    gf4_mul(gf4_add(GF4.x, GF4.zero), GF4.one) = gf4_add(gf4_mul(GF4.x, GF4.one), gf4_mul(GF4.zero, GF4.one))
    gf4_mul(gf4_add(GF4.x, GF4.zero), GF4.x) = gf4_add(gf4_mul(GF4.x, GF4.x), gf4_mul(GF4.zero, GF4.x))
    gf4_mul(gf4_add(GF4.x, GF4.zero), GF4.x_plus_one) = gf4_add(gf4_mul(GF4.x, GF4.x_plus_one), gf4_mul(GF4.zero, GF4.x_plus_one))
    w(GF4.zero)
    w(GF4.one)
    w(GF4.x)
    w(GF4.x_plus_one)
    GF4.induction(w)
    w(z)
    gf4_mul(gf4_add(GF4.x, GF4.zero), z) = gf4_add(gf4_mul(GF4.x, z), gf4_mul(GF4.zero, z))
}

/// gf4_mul_add_right with the first two arguments fixed.
theorem gf4_mul_add_right_e2e1(z: GF4) {
    gf4_mul(gf4_add(GF4.x, GF4.one), z) = gf4_add(gf4_mul(GF4.x, z), gf4_mul(GF4.one, z))
} by {
    define w(a: GF4) -> Bool {
        gf4_mul(gf4_add(GF4.x, GF4.one), a) = gf4_add(gf4_mul(GF4.x, a), gf4_mul(GF4.one, a))
    }
    gf4_mul(gf4_add(GF4.x, GF4.one), GF4.zero) = gf4_add(gf4_mul(GF4.x, GF4.zero), gf4_mul(GF4.one, GF4.zero))
    gf4_mul(gf4_add(GF4.x, GF4.one), GF4.one) = gf4_add(gf4_mul(GF4.x, GF4.one), gf4_mul(GF4.one, GF4.one))
    gf4_mul(gf4_add(GF4.x, GF4.one), GF4.x) = gf4_add(gf4_mul(GF4.x, GF4.x), gf4_mul(GF4.one, GF4.x))
    gf4_mul(gf4_add(GF4.x, GF4.one), GF4.x_plus_one) = gf4_add(gf4_mul(GF4.x, GF4.x_plus_one), gf4_mul(GF4.one, GF4.x_plus_one))
    w(GF4.zero)
    w(GF4.one)
    w(GF4.x)
    w(GF4.x_plus_one)
    GF4.induction(w)
    w(z)
    gf4_mul(gf4_add(GF4.x, GF4.one), z) = gf4_add(gf4_mul(GF4.x, z), gf4_mul(GF4.one, z))
}

/// gf4_mul_add_right with the first two arguments fixed.
theorem gf4_mul_add_right_e2e2(z: GF4) {
    gf4_mul(gf4_add(GF4.x, GF4.x), z) = gf4_add(gf4_mul(GF4.x, z), gf4_mul(GF4.x, z))
} by {
    define w(a: GF4) -> Bool {
        gf4_mul(gf4_add(GF4.x, GF4.x), a) = gf4_add(gf4_mul(GF4.x, a), gf4_mul(GF4.x, a))
    }
    gf4_mul(gf4_add(GF4.x, GF4.x), GF4.zero) = gf4_add(gf4_mul(GF4.x, GF4.zero), gf4_mul(GF4.x, GF4.zero))
    gf4_mul(gf4_add(GF4.x, GF4.x), GF4.one) = gf4_add(gf4_mul(GF4.x, GF4.one), gf4_mul(GF4.x, GF4.one))
    gf4_mul(gf4_add(GF4.x, GF4.x), GF4.x) = gf4_add(gf4_mul(GF4.x, GF4.x), gf4_mul(GF4.x, GF4.x))
    gf4_mul(gf4_add(GF4.x, GF4.x), GF4.x_plus_one) = gf4_add(gf4_mul(GF4.x, GF4.x_plus_one), gf4_mul(GF4.x, GF4.x_plus_one))
    w(GF4.zero)
    w(GF4.one)
    w(GF4.x)
    w(GF4.x_plus_one)
    GF4.induction(w)
    w(z)
    gf4_mul(gf4_add(GF4.x, GF4.x), z) = gf4_add(gf4_mul(GF4.x, z), gf4_mul(GF4.x, z))
}

/// gf4_mul_add_right with the first two arguments fixed.
theorem gf4_mul_add_right_e2e3(z: GF4) {
    gf4_mul(gf4_add(GF4.x, GF4.x_plus_one), z) = gf4_add(gf4_mul(GF4.x, z), gf4_mul(GF4.x_plus_one, z))
} by {
    define w(a: GF4) -> Bool {
        gf4_mul(gf4_add(GF4.x, GF4.x_plus_one), a) = gf4_add(gf4_mul(GF4.x, a), gf4_mul(GF4.x_plus_one, a))
    }
    gf4_mul(gf4_add(GF4.x, GF4.x_plus_one), GF4.zero) = gf4_add(gf4_mul(GF4.x, GF4.zero), gf4_mul(GF4.x_plus_one, GF4.zero))
    gf4_mul(gf4_add(GF4.x, GF4.x_plus_one), GF4.one) = gf4_add(gf4_mul(GF4.x, GF4.one), gf4_mul(GF4.x_plus_one, GF4.one))
    gf4_mul(gf4_add(GF4.x, GF4.x_plus_one), GF4.x) = gf4_add(gf4_mul(GF4.x, GF4.x), gf4_mul(GF4.x_plus_one, GF4.x))
    gf4_mul(gf4_add(GF4.x, GF4.x_plus_one), GF4.x_plus_one) = gf4_add(gf4_mul(GF4.x, GF4.x_plus_one), gf4_mul(GF4.x_plus_one, GF4.x_plus_one))
    w(GF4.zero)
    w(GF4.one)
    w(GF4.x)
    w(GF4.x_plus_one)
    GF4.induction(w)
    w(z)
    gf4_mul(gf4_add(GF4.x, GF4.x_plus_one), z) = gf4_add(gf4_mul(GF4.x, z), gf4_mul(GF4.x_plus_one, z))
}

/// gf4_mul_add_right with the first two arguments fixed.
theorem gf4_mul_add_right_e3e0(z: GF4) {
    gf4_mul(gf4_add(GF4.x_plus_one, GF4.zero), z) = gf4_add(gf4_mul(GF4.x_plus_one, z), gf4_mul(GF4.zero, z))
} by {
    define w(a: GF4) -> Bool {
        gf4_mul(gf4_add(GF4.x_plus_one, GF4.zero), a) = gf4_add(gf4_mul(GF4.x_plus_one, a), gf4_mul(GF4.zero, a))
    }
    gf4_mul(gf4_add(GF4.x_plus_one, GF4.zero), GF4.zero) = gf4_add(gf4_mul(GF4.x_plus_one, GF4.zero), gf4_mul(GF4.zero, GF4.zero))
    gf4_mul(gf4_add(GF4.x_plus_one, GF4.zero), GF4.one) = gf4_add(gf4_mul(GF4.x_plus_one, GF4.one), gf4_mul(GF4.zero, GF4.one))
    gf4_mul(gf4_add(GF4.x_plus_one, GF4.zero), GF4.x) = gf4_add(gf4_mul(GF4.x_plus_one, GF4.x), gf4_mul(GF4.zero, GF4.x))
    gf4_mul(gf4_add(GF4.x_plus_one, GF4.zero), GF4.x_plus_one) = gf4_add(gf4_mul(GF4.x_plus_one, GF4.x_plus_one), gf4_mul(GF4.zero, GF4.x_plus_one))
    w(GF4.zero)
    w(GF4.one)
    w(GF4.x)
    w(GF4.x_plus_one)
    GF4.induction(w)
    w(z)
    gf4_mul(gf4_add(GF4.x_plus_one, GF4.zero), z) = gf4_add(gf4_mul(GF4.x_plus_one, z), gf4_mul(GF4.zero, z))
}

/// gf4_mul_add_right with the first two arguments fixed.
theorem gf4_mul_add_right_e3e1(z: GF4) {
    gf4_mul(gf4_add(GF4.x_plus_one, GF4.one), z) = gf4_add(gf4_mul(GF4.x_plus_one, z), gf4_mul(GF4.one, z))
} by {
    define w(a: GF4) -> Bool {
        gf4_mul(gf4_add(GF4.x_plus_one, GF4.one), a) = gf4_add(gf4_mul(GF4.x_plus_one, a), gf4_mul(GF4.one, a))
    }
    gf4_mul(gf4_add(GF4.x_plus_one, GF4.one), GF4.zero) = gf4_add(gf4_mul(GF4.x_plus_one, GF4.zero), gf4_mul(GF4.one, GF4.zero))
    gf4_mul(gf4_add(GF4.x_plus_one, GF4.one), GF4.one) = gf4_add(gf4_mul(GF4.x_plus_one, GF4.one), gf4_mul(GF4.one, GF4.one))
    gf4_mul(gf4_add(GF4.x_plus_one, GF4.one), GF4.x) = gf4_add(gf4_mul(GF4.x_plus_one, GF4.x), gf4_mul(GF4.one, GF4.x))
    gf4_mul(gf4_add(GF4.x_plus_one, GF4.one), GF4.x_plus_one) = gf4_add(gf4_mul(GF4.x_plus_one, GF4.x_plus_one), gf4_mul(GF4.one, GF4.x_plus_one))
    w(GF4.zero)
    w(GF4.one)
    w(GF4.x)
    w(GF4.x_plus_one)
    GF4.induction(w)
    w(z)
    gf4_mul(gf4_add(GF4.x_plus_one, GF4.one), z) = gf4_add(gf4_mul(GF4.x_plus_one, z), gf4_mul(GF4.one, z))
}

/// gf4_mul_add_right with the first two arguments fixed.
theorem gf4_mul_add_right_e3e2(z: GF4) {
    gf4_mul(gf4_add(GF4.x_plus_one, GF4.x), z) = gf4_add(gf4_mul(GF4.x_plus_one, z), gf4_mul(GF4.x, z))
} by {
    define w(a: GF4) -> Bool {
        gf4_mul(gf4_add(GF4.x_plus_one, GF4.x), a) = gf4_add(gf4_mul(GF4.x_plus_one, a), gf4_mul(GF4.x, a))
    }
    gf4_mul(gf4_add(GF4.x_plus_one, GF4.x), GF4.zero) = gf4_add(gf4_mul(GF4.x_plus_one, GF4.zero), gf4_mul(GF4.x, GF4.zero))
    gf4_mul(gf4_add(GF4.x_plus_one, GF4.x), GF4.one) = gf4_add(gf4_mul(GF4.x_plus_one, GF4.one), gf4_mul(GF4.x, GF4.one))
    gf4_mul(gf4_add(GF4.x_plus_one, GF4.x), GF4.x) = gf4_add(gf4_mul(GF4.x_plus_one, GF4.x), gf4_mul(GF4.x, GF4.x))
    gf4_mul(gf4_add(GF4.x_plus_one, GF4.x), GF4.x_plus_one) = gf4_add(gf4_mul(GF4.x_plus_one, GF4.x_plus_one), gf4_mul(GF4.x, GF4.x_plus_one))
    w(GF4.zero)
    w(GF4.one)
    w(GF4.x)
    w(GF4.x_plus_one)
    GF4.induction(w)
    w(z)
    gf4_mul(gf4_add(GF4.x_plus_one, GF4.x), z) = gf4_add(gf4_mul(GF4.x_plus_one, z), gf4_mul(GF4.x, z))
}

/// gf4_mul_add_right with the first two arguments fixed.
theorem gf4_mul_add_right_e3e3(z: GF4) {
    gf4_mul(gf4_add(GF4.x_plus_one, GF4.x_plus_one), z) = gf4_add(gf4_mul(GF4.x_plus_one, z), gf4_mul(GF4.x_plus_one, z))
} by {
    define w(a: GF4) -> Bool {
        gf4_mul(gf4_add(GF4.x_plus_one, GF4.x_plus_one), a) = gf4_add(gf4_mul(GF4.x_plus_one, a), gf4_mul(GF4.x_plus_one, a))
    }
    gf4_mul(gf4_add(GF4.x_plus_one, GF4.x_plus_one), GF4.zero) = gf4_add(gf4_mul(GF4.x_plus_one, GF4.zero), gf4_mul(GF4.x_plus_one, GF4.zero))
    gf4_mul(gf4_add(GF4.x_plus_one, GF4.x_plus_one), GF4.one) = gf4_add(gf4_mul(GF4.x_plus_one, GF4.one), gf4_mul(GF4.x_plus_one, GF4.one))
    gf4_mul(gf4_add(GF4.x_plus_one, GF4.x_plus_one), GF4.x) = gf4_add(gf4_mul(GF4.x_plus_one, GF4.x), gf4_mul(GF4.x_plus_one, GF4.x))
    gf4_mul(gf4_add(GF4.x_plus_one, GF4.x_plus_one), GF4.x_plus_one) = gf4_add(gf4_mul(GF4.x_plus_one, GF4.x_plus_one), gf4_mul(GF4.x_plus_one, GF4.x_plus_one))
    w(GF4.zero)
    w(GF4.one)
    w(GF4.x)
    w(GF4.x_plus_one)
    GF4.induction(w)
    w(z)
    gf4_mul(gf4_add(GF4.x_plus_one, GF4.x_plus_one), z) = gf4_add(gf4_mul(GF4.x_plus_one, z), gf4_mul(GF4.x_plus_one, z))
}

/// gf4_mul_add_right with the first argument fixed at GF4.zero.
theorem gf4_mul_add_right_e0(y: GF4, z: GF4) {
    gf4_mul(gf4_add(GF4.zero, y), z) = gf4_add(gf4_mul(GF4.zero, z), gf4_mul(y, z))
} by {
    gf4_cases(y)
    if y = GF4.zero {
        gf4_mul_add_right_e0e0(z)
        gf4_mul(gf4_add(GF4.zero, y), z) = gf4_add(gf4_mul(GF4.zero, z), gf4_mul(y, z))
    }
    if y = GF4.one {
        gf4_mul_add_right_e0e1(z)
        gf4_mul(gf4_add(GF4.zero, y), z) = gf4_add(gf4_mul(GF4.zero, z), gf4_mul(y, z))
    }
    if y = GF4.x {
        gf4_mul_add_right_e0e2(z)
        gf4_mul(gf4_add(GF4.zero, y), z) = gf4_add(gf4_mul(GF4.zero, z), gf4_mul(y, z))
    }
    if y = GF4.x_plus_one {
        gf4_mul_add_right_e0e3(z)
        gf4_mul(gf4_add(GF4.zero, y), z) = gf4_add(gf4_mul(GF4.zero, z), gf4_mul(y, z))
    }
    gf4_mul(gf4_add(GF4.zero, y), z) = gf4_add(gf4_mul(GF4.zero, z), gf4_mul(y, z))
}

/// gf4_mul_add_right with the first argument fixed at GF4.one.
theorem gf4_mul_add_right_e1(y: GF4, z: GF4) {
    gf4_mul(gf4_add(GF4.one, y), z) = gf4_add(gf4_mul(GF4.one, z), gf4_mul(y, z))
} by {
    gf4_cases(y)
    if y = GF4.zero {
        gf4_mul_add_right_e1e0(z)
        gf4_mul(gf4_add(GF4.one, y), z) = gf4_add(gf4_mul(GF4.one, z), gf4_mul(y, z))
    }
    if y = GF4.one {
        gf4_mul_add_right_e1e1(z)
        gf4_mul(gf4_add(GF4.one, y), z) = gf4_add(gf4_mul(GF4.one, z), gf4_mul(y, z))
    }
    if y = GF4.x {
        gf4_mul_add_right_e1e2(z)
        gf4_mul(gf4_add(GF4.one, y), z) = gf4_add(gf4_mul(GF4.one, z), gf4_mul(y, z))
    }
    if y = GF4.x_plus_one {
        gf4_mul_add_right_e1e3(z)
        gf4_mul(gf4_add(GF4.one, y), z) = gf4_add(gf4_mul(GF4.one, z), gf4_mul(y, z))
    }
    gf4_mul(gf4_add(GF4.one, y), z) = gf4_add(gf4_mul(GF4.one, z), gf4_mul(y, z))
}

/// gf4_mul_add_right with the first argument fixed at GF4.x.
theorem gf4_mul_add_right_e2(y: GF4, z: GF4) {
    gf4_mul(gf4_add(GF4.x, y), z) = gf4_add(gf4_mul(GF4.x, z), gf4_mul(y, z))
} by {
    gf4_cases(y)
    if y = GF4.zero {
        gf4_mul_add_right_e2e0(z)
        gf4_mul(gf4_add(GF4.x, y), z) = gf4_add(gf4_mul(GF4.x, z), gf4_mul(y, z))
    }
    if y = GF4.one {
        gf4_mul_add_right_e2e1(z)
        gf4_mul(gf4_add(GF4.x, y), z) = gf4_add(gf4_mul(GF4.x, z), gf4_mul(y, z))
    }
    if y = GF4.x {
        gf4_mul_add_right_e2e2(z)
        gf4_mul(gf4_add(GF4.x, y), z) = gf4_add(gf4_mul(GF4.x, z), gf4_mul(y, z))
    }
    if y = GF4.x_plus_one {
        gf4_mul_add_right_e2e3(z)
        gf4_mul(gf4_add(GF4.x, y), z) = gf4_add(gf4_mul(GF4.x, z), gf4_mul(y, z))
    }
    gf4_mul(gf4_add(GF4.x, y), z) = gf4_add(gf4_mul(GF4.x, z), gf4_mul(y, z))
}

/// gf4_mul_add_right with the first argument fixed at GF4.x_plus_one.
theorem gf4_mul_add_right_e3(y: GF4, z: GF4) {
    gf4_mul(gf4_add(GF4.x_plus_one, y), z) = gf4_add(gf4_mul(GF4.x_plus_one, z), gf4_mul(y, z))
} by {
    gf4_cases(y)
    if y = GF4.zero {
        gf4_mul_add_right_e3e0(z)
        gf4_mul(gf4_add(GF4.x_plus_one, y), z) = gf4_add(gf4_mul(GF4.x_plus_one, z), gf4_mul(y, z))
    }
    if y = GF4.one {
        gf4_mul_add_right_e3e1(z)
        gf4_mul(gf4_add(GF4.x_plus_one, y), z) = gf4_add(gf4_mul(GF4.x_plus_one, z), gf4_mul(y, z))
    }
    if y = GF4.x {
        gf4_mul_add_right_e3e2(z)
        gf4_mul(gf4_add(GF4.x_plus_one, y), z) = gf4_add(gf4_mul(GF4.x_plus_one, z), gf4_mul(y, z))
    }
    if y = GF4.x_plus_one {
        gf4_mul_add_right_e3e3(z)
        gf4_mul(gf4_add(GF4.x_plus_one, y), z) = gf4_add(gf4_mul(GF4.x_plus_one, z), gf4_mul(y, z))
    }
    gf4_mul(gf4_add(GF4.x_plus_one, y), z) = gf4_add(gf4_mul(GF4.x_plus_one, z), gf4_mul(y, z))
}

/// gf4_mul_add_right.
theorem gf4_mul_add_right(x: GF4, y: GF4, z: GF4) {
    Mul.mul(Add.add(x, y), z) = Add.add(Mul.mul(x, z), Mul.mul(y, z))
} by {
    gf4_cases(x)
    if x = GF4.zero {
        gf4_mul_add_right_e0(y, z)
        gf4_mul(gf4_add(GF4.zero, y), z) = gf4_add(gf4_mul(GF4.zero, z), gf4_mul(y, z))
        gf4_mul(gf4_add(x, y), z) = gf4_add(gf4_mul(x, z), gf4_mul(y, z))
    }
    if x = GF4.one {
        gf4_mul_add_right_e1(y, z)
        gf4_mul(gf4_add(GF4.one, y), z) = gf4_add(gf4_mul(GF4.one, z), gf4_mul(y, z))
        gf4_mul(gf4_add(x, y), z) = gf4_add(gf4_mul(x, z), gf4_mul(y, z))
    }
    if x = GF4.x {
        gf4_mul_add_right_e2(y, z)
        gf4_mul(gf4_add(GF4.x, y), z) = gf4_add(gf4_mul(GF4.x, z), gf4_mul(y, z))
        gf4_mul(gf4_add(x, y), z) = gf4_add(gf4_mul(x, z), gf4_mul(y, z))
    }
    if x = GF4.x_plus_one {
        gf4_mul_add_right_e3(y, z)
        gf4_mul(gf4_add(GF4.x_plus_one, y), z) = gf4_add(gf4_mul(GF4.x_plus_one, z), gf4_mul(y, z))
        gf4_mul(gf4_add(x, y), z) = gf4_add(gf4_mul(x, z), gf4_mul(y, z))
    }
    gf4_mul(gf4_add(x, y), z) = gf4_add(gf4_mul(x, z), gf4_mul(y, z))
    Mul.mul(Add.add(x, y), z) = Add.add(Mul.mul(x, z), Mul.mul(y, z))
}

/// The inverse of zero is zero in GF4.
theorem gf4_inverse_zero {
    Zero.0[GF4].inverse = Zero.0[GF4]
} by {
    gf4_inv(GF4.zero) = GF4.zero
    Zero.0[GF4].inverse = gf4_inv(Zero.0[GF4])
    Zero.0[GF4].inverse = GF4.zero
    Zero.0[GF4].inverse = Zero.0[GF4]
}

/// Every nonzero element of GF4 has a multiplicative inverse.
theorem gf4_field_inverse(x: GF4) {
    x != Zero.0[GF4] implies x * x.inverse = One.1[GF4]
} by {
    define w(a: GF4) -> Bool {
        a != GF4.zero implies gf4_mul(a, gf4_inv(a)) = GF4.one
    }
    if GF4.zero != GF4.zero {
        false
    }
    gf4_mul(GF4.one, gf4_inv(GF4.one)) = GF4.one
    gf4_mul(GF4.x, gf4_inv(GF4.x)) = GF4.one
    gf4_mul(GF4.x_plus_one, gf4_inv(GF4.x_plus_one)) = GF4.one
    w(GF4.zero)
    w(GF4.one)
    w(GF4.x)
    w(GF4.x_plus_one)
    GF4.induction(w)
    w(x)
    if x != Zero.0[GF4] {
        x != GF4.zero
        gf4_mul(x, gf4_inv(x)) = GF4.one
        gf4_mul(x, x.inverse) = GF4.one
        x * x.inverse = GF4.one
        x * x.inverse = One.1[GF4]
    }
}

/// Zero and one are distinct in GF4.
theorem gf4_zero_ne_one {
    Zero.0[GF4] != One.1[GF4]
} by {
    GF4.zero != GF4.one
    Zero.0[GF4] != One.1[GF4]
}

/// The elements of GF4 form an additive semigroup.
instance GF4: AddSemigroup

/// The elements of GF4 form an additive commutative semigroup.
instance GF4: AddCommSemigroup

/// The elements of GF4 form an additive monoid.
instance GF4: AddMonoid

/// The elements of GF4 form an additive commutative monoid.
instance GF4: AddCommMonoid

/// The elements of GF4 form an additive group.
instance GF4: AddGroup

/// The elements of GF4 form an additive commutative group.
instance GF4: AddCommGroup

/// The elements of GF4 form a multiplicative semigroup.
instance GF4: Semigroup

/// The elements of GF4 form a multiplicative commutative semigroup.
instance GF4: CommSemigroup

/// The elements of GF4 form a multiplicative monoid.
instance GF4: Monoid

/// The elements of GF4 form a multiplicative commutative monoid.
instance GF4: CommMonoid

/// The elements of GF4 form a semiring.
instance GF4: Semiring

/// The elements of GF4 form a ring.
instance GF4: Ring

/// The elements of GF4 form a commutative ring.
instance GF4: CommRing

/// The elements of GF4 form a field.
instance GF4: Field

/// The defining relation of GF(4): the class of X satisfies X^2 + X + 1 = 0.
theorem gf4_x_squared_plus_x_plus_one_eq_zero {
    gf4_add(gf4_add(gf4_mul(GF4.x, GF4.x), GF4.x), GF4.one) = GF4.zero
} by {
    gf4_add(gf4_add(gf4_mul(GF4.x, GF4.x), GF4.x), GF4.one) = GF4.zero
}

/// The class of X is a unit with inverse X + 1: X * (X + 1) = 1.
theorem gf4_x_inverse {
    gf4_mul(GF4.x, GF4.x_plus_one) = GF4.one
} by {
    gf4_mul(GF4.x, GF4.x_plus_one) = GF4.one
}

/// The class of X + 1 is a unit with inverse X: (X + 1) * X = 1.
theorem gf4_x_plus_one_inverse {
    gf4_mul(GF4.x_plus_one, GF4.x) = GF4.one
} by {
    gf4_mul(GF4.x_plus_one, GF4.x) = GF4.one
}

/// The four elements of GF(4) are pairwise distinct: 0 and 1 differ.
theorem gf4_zero_ne_one_elt {
    GF4.zero != GF4.one
} by {
    GF4.zero != GF4.one
}

/// The four elements of GF(4) are pairwise distinct: 0 and X differ.
theorem gf4_zero_ne_x {
    GF4.zero != GF4.x
} by {
    GF4.zero != GF4.x
}

/// The four elements of GF(4) are pairwise distinct: 0 and X + 1 differ.
theorem gf4_zero_ne_x_plus_one {
    GF4.zero != GF4.x_plus_one
} by {
    GF4.zero != GF4.x_plus_one
}

/// The four elements of GF(4) are pairwise distinct: 1 and X differ.
theorem gf4_one_ne_x {
    GF4.one != GF4.x
} by {
    GF4.one != GF4.x
}

/// The four elements of GF(4) are pairwise distinct: 1 and X + 1 differ.
theorem gf4_one_ne_x_plus_one {
    GF4.one != GF4.x_plus_one
} by {
    GF4.one != GF4.x_plus_one
}

/// The four elements of GF(4) are pairwise distinct: X and X + 1 differ.
theorem gf4_x_ne_x_plus_one {
    GF4.x != GF4.x_plus_one
} by {
    GF4.x != GF4.x_plus_one
}
