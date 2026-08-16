/// The third isomorphism theorem: if `N ≤ M ≤ G` with both `N` and `M`
/// normal in `G`, then `(G/N)/(M/N) ≅ G/M`.  The induced map `G/N -> G/M`
/// sends the `N`-coset of `g` to the `M`-coset of `g`; its kernel is
/// `M/N` (elements of `G/N` whose representatives lie in `M`), and it is
/// surjective onto `G/M`.
///
/// As in `second_iso`, quotient groups are not bundled `Group` instances, so
/// the isomorphism content is stated at the level of quotient projections.

from algebra.group import Group
from algebra.subgroup import Subgroup, subgroup_subset, subgroup_mul_mem,
    subgroup_inv_mem, subgroup_contains_identity
from algebra.group.normal_subgroup import is_normal_subgroup, normal_conjugate_mem,
    normal_subgroup_rel, normal_subgroup_quotient_project,
    normal_subgroup_quotient_project_eq_mk, normal_subgroup_quotient_project_eq_of_rel,
    normal_subgroup_quotient_project_eq_iff_rel, normal_subgroup_quotient_relation,
    normal_subgroup_quotient_relation_rel, normal_subgroup_quotient_one,
    normal_subgroup_quotient_project_one, normal_subgroup_quotient_mul,
    normal_subgroup_quotient_project_mul, normal_subgroup_quotient_inverse,
    normal_subgroup_quotient_project_inverse, normal_subgroup_quotient_project_eq_one_iff_contains
from data.basic.equivalence import QuotientOver, quotient_over_lift_to,
    quotient_over_lift_to_projection, quotient_respects, quotient_over_mk,
    quotient_over_mk_surjective, quotient_over_mk_fields

// ==== The induced map G/N -> G/M ====

/// Projection to `G/M` as a plain representative function on `G`.
define third_iso_projection[G: Group](m: Subgroup[G], g: G) -> QuotientOver[G] {
    normal_subgroup_quotient_project(m, g)
}

/// Projection to `G/M` collapses `N`-cosets when `N` is contained in `M`.
theorem third_iso_well_defined[G: Group](n: Subgroup[G], m: Subgroup[G], a: G, b: G) {
    subgroup_subset(n, m) and normal_subgroup_rel(n, a, b) implies
    normal_subgroup_quotient_project(m, a) = normal_subgroup_quotient_project(m, b)
} by {
    if subgroup_subset(n, m) and normal_subgroup_rel(n, a, b) {
        normal_subgroup_rel(n, a, b) = n.contains(a * b.inverse)
        n.contains(a * b.inverse)
        subgroup_subset(n, m) = forall(x: G) {
            n.contains(x) implies m.contains(x)
        }
        m.contains(a * b.inverse)
        normal_subgroup_rel(m, a, b) = m.contains(a * b.inverse)
        normal_subgroup_rel(m, a, b)
        normal_subgroup_quotient_project_eq_of_rel(m, a, b)
        normal_subgroup_quotient_project(m, a) = normal_subgroup_quotient_project(m, b)
    }
}

/// The projection to `G/M` respects the `N`-quotient relation when `N ≤ M`.
theorem third_iso_projection_respects[G: Group](n: Subgroup[G], m: Subgroup[G]) {
    subgroup_subset(n, m) implies
    quotient_respects(normal_subgroup_quotient_relation(n), third_iso_projection(m))
} by {
    if subgroup_subset(n, m) {
        forall(a: G, b: G) {
            if normal_subgroup_quotient_relation(n).rel(a, b) {
                normal_subgroup_quotient_relation_rel(n)
                normal_subgroup_quotient_relation(n).rel(a, b) = normal_subgroup_rel(n, a, b)
                normal_subgroup_rel(n, a, b)
                third_iso_well_defined(n, m, a, b)
                normal_subgroup_quotient_project(m, a) = normal_subgroup_quotient_project(m, b)
                third_iso_projection(m, a) = normal_subgroup_quotient_project(m, a)
                third_iso_projection(m, b) = normal_subgroup_quotient_project(m, b)
                third_iso_projection(m, a) = third_iso_projection(m, b)
            }
        }
        quotient_respects(normal_subgroup_quotient_relation(n), third_iso_projection(m))
    }
}

/// The map `G/N -> G/M` induced by projection to `G/M`.
define third_iso_map[G: Group](n: Subgroup[G], m: Subgroup[G]) -> QuotientOver[G] -> QuotientOver[G] {
    quotient_over_lift_to(third_iso_projection(m))
}

/// The induced map agrees with the projection to `G/M` on canonical `N`-cosets.
theorem third_iso_map_project[G: Group](n: Subgroup[G], m: Subgroup[G], g: G) {
    subgroup_subset(n, m) implies
    third_iso_map(n, m, normal_subgroup_quotient_project(n, g)) =
        normal_subgroup_quotient_project(m, g)
} by {
    if subgroup_subset(n, m) {
        normal_subgroup_quotient_project_eq_mk(n, g)
        normal_subgroup_quotient_relation_rel(n)
        third_iso_projection_respects(n, m)
        quotient_over_lift_to_projection(normal_subgroup_quotient_relation(n),
            third_iso_projection(m), g)
        quotient_over_lift_to(third_iso_projection(m),
            quotient_over_mk(normal_subgroup_quotient_relation(n), g)) =
            third_iso_projection(m, g)
        third_iso_projection(m, g) = normal_subgroup_quotient_project(m, g)
        third_iso_map(n, m, quotient_over_mk(normal_subgroup_quotient_relation(n), g)) =
            normal_subgroup_quotient_project(m, g)
        third_iso_map(n, m, normal_subgroup_quotient_project(n, g)) =
            normal_subgroup_quotient_project(m, g)
    }
}

/// The induced map `G/N -> G/M` preserves multiplication of canonical cosets.
theorem third_iso_map_mul[G: Group](n: Subgroup[G], m: Subgroup[G], a: G, b: G) {
    is_normal_subgroup(n) and is_normal_subgroup(m) and subgroup_subset(n, m) implies
    third_iso_map(n, m,
        normal_subgroup_quotient_mul(n,
            normal_subgroup_quotient_project(n, a),
            normal_subgroup_quotient_project(n, b))) =
        normal_subgroup_quotient_mul(m,
            third_iso_map(n, m, normal_subgroup_quotient_project(n, a)),
            third_iso_map(n, m, normal_subgroup_quotient_project(n, b)))
} by {
    if is_normal_subgroup(n) and is_normal_subgroup(m) and subgroup_subset(n, m) {
        normal_subgroup_quotient_project_mul(n, a, b)
        normal_subgroup_quotient_project(n, a * b) =
            normal_subgroup_quotient_mul(n,
                normal_subgroup_quotient_project(n, a),
                normal_subgroup_quotient_project(n, b))
        third_iso_map_project(n, m, a * b)
        third_iso_map(n, m, normal_subgroup_quotient_project(n, a * b)) =
            normal_subgroup_quotient_project(m, a * b)
        third_iso_map(n, m,
            normal_subgroup_quotient_mul(n,
                normal_subgroup_quotient_project(n, a),
                normal_subgroup_quotient_project(n, b))) =
            normal_subgroup_quotient_project(m, a * b)
        normal_subgroup_quotient_project_mul(m, a, b)
        normal_subgroup_quotient_project(m, a * b) =
            normal_subgroup_quotient_mul(m,
                normal_subgroup_quotient_project(m, a),
                normal_subgroup_quotient_project(m, b))
        third_iso_map_project(n, m, a)
        third_iso_map(n, m, normal_subgroup_quotient_project(n, a)) =
            normal_subgroup_quotient_project(m, a)
        third_iso_map_project(n, m, b)
        third_iso_map(n, m, normal_subgroup_quotient_project(n, b)) =
            normal_subgroup_quotient_project(m, b)
        normal_subgroup_quotient_mul(m,
            normal_subgroup_quotient_project(m, a),
            normal_subgroup_quotient_project(m, b)) =
            normal_subgroup_quotient_mul(m,
                third_iso_map(n, m, normal_subgroup_quotient_project(n, a)),
                third_iso_map(n, m, normal_subgroup_quotient_project(n, b)))
        normal_subgroup_quotient_project(m, a * b) =
            normal_subgroup_quotient_mul(m,
                third_iso_map(n, m, normal_subgroup_quotient_project(n, a)),
                third_iso_map(n, m, normal_subgroup_quotient_project(n, b)))
        third_iso_map(n, m,
            normal_subgroup_quotient_mul(n,
                normal_subgroup_quotient_project(n, a),
                normal_subgroup_quotient_project(n, b))) =
            normal_subgroup_quotient_mul(m,
                third_iso_map(n, m, normal_subgroup_quotient_project(n, a)),
                third_iso_map(n, m, normal_subgroup_quotient_project(n, b)))
    }
}

/// The induced map sends the `N`-quotient identity to the `M`-quotient identity.
theorem third_iso_map_one[G: Group](n: Subgroup[G], m: Subgroup[G]) {
    subgroup_subset(n, m) implies
    third_iso_map(n, m, normal_subgroup_quotient_one(n)) = normal_subgroup_quotient_one(m)
} by {
    if subgroup_subset(n, m) {
        normal_subgroup_quotient_project_one(n)
        third_iso_map_project(n, m, G.1)
        third_iso_map(n, m, normal_subgroup_quotient_project(n, G.1)) =
            normal_subgroup_quotient_project(m, G.1)
        normal_subgroup_quotient_project_one(m)
        third_iso_map(n, m, normal_subgroup_quotient_one(n)) = normal_subgroup_quotient_one(m)
    }
}

/// The induced map `G/N -> G/M` preserves inversion of canonical cosets.
theorem third_iso_map_inverse[G: Group](n: Subgroup[G], m: Subgroup[G], a: G) {
    is_normal_subgroup(n) and is_normal_subgroup(m) and subgroup_subset(n, m) implies
    third_iso_map(n, m,
        normal_subgroup_quotient_inverse(n, normal_subgroup_quotient_project(n, a))) =
        normal_subgroup_quotient_inverse(m,
            third_iso_map(n, m, normal_subgroup_quotient_project(n, a)))
} by {
    if is_normal_subgroup(n) and is_normal_subgroup(m) and subgroup_subset(n, m) {
        normal_subgroup_quotient_project_inverse(n, a)
        normal_subgroup_quotient_project(n, a.inverse) =
            normal_subgroup_quotient_inverse(n, normal_subgroup_quotient_project(n, a))
        third_iso_map_project(n, m, a.inverse)
        third_iso_map(n, m, normal_subgroup_quotient_project(n, a.inverse)) =
            normal_subgroup_quotient_project(m, a.inverse)
        third_iso_map(n, m,
            normal_subgroup_quotient_inverse(n, normal_subgroup_quotient_project(n, a))) =
            normal_subgroup_quotient_project(m, a.inverse)
        normal_subgroup_quotient_project_inverse(m, a)
        normal_subgroup_quotient_project(m, a.inverse) =
            normal_subgroup_quotient_inverse(m, normal_subgroup_quotient_project(m, a))
        third_iso_map_project(n, m, a)
        third_iso_map(n, m, normal_subgroup_quotient_project(n, a)) =
            normal_subgroup_quotient_project(m, a)
        normal_subgroup_quotient_project(m, a.inverse) =
            normal_subgroup_quotient_inverse(m,
                third_iso_map(n, m, normal_subgroup_quotient_project(n, a)))
        third_iso_map(n, m,
            normal_subgroup_quotient_inverse(n, normal_subgroup_quotient_project(n, a))) =
            normal_subgroup_quotient_inverse(m,
                third_iso_map(n, m, normal_subgroup_quotient_project(n, a)))
    }
}

// ==== The kernel of the induced map is M/N ====

/// The kernel of the induced map `G/N -> G/M` is exactly `M/N`: two elements
/// of `G` have equal induced images exactly when they differ by an element
/// of `M`.
theorem third_iso_kernel_iff[G: Group](n: Subgroup[G], m: Subgroup[G], a: G, b: G) {
    subgroup_subset(n, m) implies
    (third_iso_map(n, m, normal_subgroup_quotient_project(n, a)) =
        third_iso_map(n, m, normal_subgroup_quotient_project(n, b))) =
        m.contains(a * b.inverse)
} by {
    if subgroup_subset(n, m) {
        third_iso_map_project(n, m, a)
        third_iso_map(n, m, normal_subgroup_quotient_project(n, a)) =
            normal_subgroup_quotient_project(m, a)
        third_iso_map_project(n, m, b)
        third_iso_map(n, m, normal_subgroup_quotient_project(n, b)) =
            normal_subgroup_quotient_project(m, b)
        normal_subgroup_quotient_project_eq_iff_rel(m, a, b)
        (normal_subgroup_quotient_project(m, a) = normal_subgroup_quotient_project(m, b)) =
            normal_subgroup_rel(m, a, b)
        normal_subgroup_rel(m, a, b) = m.contains(a * b.inverse)
        (third_iso_map(n, m, normal_subgroup_quotient_project(n, a)) =
            third_iso_map(n, m, normal_subgroup_quotient_project(n, b))) =
            m.contains(a * b.inverse)
    }
}

/// Two elements of `G/N` are identified by the induced map exactly when they
/// differ by an element of `M/N`: the kernel of the induced map, as a relation
/// on `G/N`, is the quotient relation of `M/N`.
theorem third_iso_kernel_quotient_rel[G: Group](n: Subgroup[G], m: Subgroup[G], a: G, b: G) {
    is_normal_subgroup(n) and subgroup_subset(n, m) implies
    (third_iso_map(n, m, normal_subgroup_quotient_project(n, a)) =
        third_iso_map(n, m, normal_subgroup_quotient_project(n, b))) =
        exists(t: G) {
            m.contains(t) and n.contains(a * b.inverse * t.inverse)
        }
} by {
    if is_normal_subgroup(n) and subgroup_subset(n, m) {
        third_iso_kernel_iff(n, m, a, b)
        (third_iso_map(n, m, normal_subgroup_quotient_project(n, a)) =
            third_iso_map(n, m, normal_subgroup_quotient_project(n, b))) =
            m.contains(a * b.inverse)
        if m.contains(a * b.inverse) {
            subgroup_contains_identity(n)
            a * b.inverse * (a * b.inverse).inverse = G.1
            n.contains(a * b.inverse * (a * b.inverse).inverse)
            exists(t: G) {
                m.contains(t) and n.contains(a * b.inverse * t.inverse)
            }
        }
        if exists(t: G) {
            m.contains(t) and n.contains(a * b.inverse * t.inverse)
        } {
            let t: G satisfy {
                m.contains(t) and n.contains(a * b.inverse * t.inverse)
            }
            n.contains(a * b.inverse * t.inverse)
            subgroup_subset(n, m) = forall(x: G) {
                n.contains(x) implies m.contains(x)
            }
            m.contains(a * b.inverse * t.inverse)
            subgroup_mul_mem(m, a * b.inverse * t.inverse, t)
            (a * b.inverse * t.inverse) * t = a * b.inverse
            m.contains(a * b.inverse)
        }
        m.contains(a * b.inverse) = exists(t: G) {
            m.contains(t) and n.contains(a * b.inverse * t.inverse)
        }
        (third_iso_map(n, m, normal_subgroup_quotient_project(n, a)) =
            third_iso_map(n, m, normal_subgroup_quotient_project(n, b))) =
            exists(t: G) {
                m.contains(t) and n.contains(a * b.inverse * t.inverse)
            }
    }
}

/// The induced map `G/N -> G/M` is surjective: every `M`-coset is the image
/// of the induced map on the corresponding `N`-coset.
theorem third_iso_surjective[G: Group](n: Subgroup[G], m: Subgroup[G], q: QuotientOver[G]) {
    subgroup_subset(n, m) and q.qrel = normal_subgroup_quotient_relation(m) implies
    exists(qp: QuotientOver[G]) {
        qp.qrel = normal_subgroup_quotient_relation(n) and third_iso_map(n, m, qp) = q
    }
} by {
    if subgroup_subset(n, m) and q.qrel = normal_subgroup_quotient_relation(m) {
        quotient_over_mk_surjective(q)
        let g: G satisfy {
            quotient_over_mk(q.qrel, g) = q
        }
        q.qrel = normal_subgroup_quotient_relation(m)
        quotient_over_mk(normal_subgroup_quotient_relation(m), g) = q
        normal_subgroup_quotient_project_eq_mk(m, g)
        normal_subgroup_quotient_project(m, g) = q
        third_iso_map_project(n, m, g)
        third_iso_map(n, m, normal_subgroup_quotient_project(n, g)) =
            normal_subgroup_quotient_project(m, g)
        third_iso_map(n, m, normal_subgroup_quotient_project(n, g)) = q
        normal_subgroup_quotient_project_eq_mk(n, g)
        quotient_over_mk_fields(normal_subgroup_quotient_relation(n), g)
        normal_subgroup_quotient_project(n, g).qrel = normal_subgroup_quotient_relation(n)
        exists(qp: QuotientOver[G]) {
            qp.qrel = normal_subgroup_quotient_relation(n) and third_iso_map(n, m, qp) = q
        }
    }
}

// ==== M/N as a subgroup of G/N ====

/// The multiplication-closure part of a subgroup of the quotient `G/N`.
define quotient_subgroup_mul_closed[G: Group](n: Subgroup[G], contains: QuotientOver[G] -> Bool) -> Bool {
    forall(a: QuotientOver[G], b: QuotientOver[G]) {
        contains(a) and contains(b) implies
        contains(normal_subgroup_quotient_mul(n, a, b))
    }
}

/// The inversion-closure part of a subgroup of the quotient `G/N`.
define quotient_subgroup_inv_closed[G: Group](n: Subgroup[G], contains: QuotientOver[G] -> Bool) -> Bool {
    forall(a: QuotientOver[G]) {
        contains(a) implies contains(normal_subgroup_quotient_inverse(n, a))
    }
}

/// True if a predicate on the quotient `G/N` is a subgroup of the quotient:
/// it contains the quotient identity and is closed under quotient
/// multiplication and inversion.
define quotient_subgroup_constraint[G: Group](n: Subgroup[G], contains: QuotientOver[G] -> Bool) -> Bool {
    contains(normal_subgroup_quotient_one(n)) and
    quotient_subgroup_mul_closed(n, contains) and
    quotient_subgroup_inv_closed(n, contains)
}

/// Membership in `M/N` as a subset of the quotient `G/N`: the quotient
/// element has a representative in `M`.
define m_over_n_contains[G: Group](n: Subgroup[G], m: Subgroup[G], q: QuotientOver[G]) -> Bool {
    exists(a: G) {
        m.contains(a) and normal_subgroup_quotient_project(n, a) = q
    }
}

/// `M/N` contains the quotient identity.
theorem m_over_n_identity[G: Group](n: Subgroup[G], m: Subgroup[G]) {
    m_over_n_contains(n, m, normal_subgroup_quotient_one(n))
} by {
    subgroup_contains_identity(m)
    normal_subgroup_quotient_project_one(n)
    normal_subgroup_quotient_project(n, G.1) = normal_subgroup_quotient_one(n)
    m_over_n_contains(n, m, normal_subgroup_quotient_one(n))
}

/// `M/N` is closed under quotient multiplication.
theorem m_over_n_mul[G: Group](n: Subgroup[G], m: Subgroup[G], q1: QuotientOver[G], q2: QuotientOver[G]) {
    is_normal_subgroup(n) and m_over_n_contains(n, m, q1) and m_over_n_contains(n, m, q2) implies
    m_over_n_contains(n, m, normal_subgroup_quotient_mul(n, q1, q2))
} by {
    if is_normal_subgroup(n) and m_over_n_contains(n, m, q1) and m_over_n_contains(n, m, q2) {
        m_over_n_contains(n, m, q1) and m_over_n_contains(n, m, q2)
        m_over_n_contains(n, m, q1)
        m_over_n_contains(n, m, q2)
        m_over_n_contains(n, m, q1) = exists(a: G) {
            m.contains(a) and normal_subgroup_quotient_project(n, a) = q1
        }
        m_over_n_contains(n, m, q2) = exists(a: G) {
            m.contains(a) and normal_subgroup_quotient_project(n, a) = q2
        }
        let a: G satisfy {
            m.contains(a) and normal_subgroup_quotient_project(n, a) = q1
        }
        let b: G satisfy {
            m.contains(b) and normal_subgroup_quotient_project(n, b) = q2
        }
        subgroup_mul_mem(m, a, b)
        normal_subgroup_quotient_project_mul(n, a, b)
        normal_subgroup_quotient_project(n, a * b) =
            normal_subgroup_quotient_mul(n,
                normal_subgroup_quotient_project(n, a),
                normal_subgroup_quotient_project(n, b))
        normal_subgroup_quotient_mul(n, q1, q2) =
            normal_subgroup_quotient_mul(n,
                normal_subgroup_quotient_project(n, a),
                normal_subgroup_quotient_project(n, b))
        normal_subgroup_quotient_project(n, a * b) =
            normal_subgroup_quotient_mul(n, q1, q2)
        m_over_n_contains(n, m, normal_subgroup_quotient_mul(n, q1, q2))
    }
}

/// `M/N` is closed under quotient inversion.
theorem m_over_n_inverse[G: Group](n: Subgroup[G], m: Subgroup[G], q: QuotientOver[G]) {
    is_normal_subgroup(n) and m_over_n_contains(n, m, q) implies
    m_over_n_contains(n, m, normal_subgroup_quotient_inverse(n, q))
} by {
    if is_normal_subgroup(n) and m_over_n_contains(n, m, q) {
        let a: G satisfy {
            m.contains(a) and normal_subgroup_quotient_project(n, a) = q
        }
        subgroup_inv_mem(m, a)
        normal_subgroup_quotient_project_inverse(n, a)
        normal_subgroup_quotient_project(n, a.inverse) =
            normal_subgroup_quotient_inverse(n, normal_subgroup_quotient_project(n, a))
        normal_subgroup_quotient_inverse(n, q) =
            normal_subgroup_quotient_inverse(n, normal_subgroup_quotient_project(n, a))
        normal_subgroup_quotient_project(n, a.inverse) =
            normal_subgroup_quotient_inverse(n, q)
        m_over_n_contains(n, m, normal_subgroup_quotient_inverse(n, q))
    }
}

/// `M/N` is a subgroup of the quotient `G/N`.
theorem m_over_n_constraint[G: Group](n: Subgroup[G], m: Subgroup[G]) {
    is_normal_subgroup(n) implies quotient_subgroup_constraint(n, m_over_n_contains(n, m))
} by {
    if is_normal_subgroup(n) {
        m_over_n_identity(n, m)
        m_over_n_contains(n, m, normal_subgroup_quotient_one(n))
        forall(q1: QuotientOver[G], q2: QuotientOver[G]) {
            if m_over_n_contains(n, m, q1) and m_over_n_contains(n, m, q2) {
                m_over_n_mul(n, m, q1, q2)
                m_over_n_contains(n, m, normal_subgroup_quotient_mul(n, q1, q2))
            }
        }
        quotient_subgroup_mul_closed(n, m_over_n_contains(n, m))
        forall(q: QuotientOver[G]) {
            if m_over_n_contains(n, m, q) {
                m_over_n_inverse(n, m, q)
                m_over_n_contains(n, m, normal_subgroup_quotient_inverse(n, q))
            }
        }
        quotient_subgroup_inv_closed(n, m_over_n_contains(n, m))
        quotient_subgroup_constraint(n, m_over_n_contains(n, m))
    }
}

/// `M/N` is closed under conjugation by canonical quotient elements: it is
/// normal in `G/N` when `M` is normal in `G`.
theorem m_over_n_normal[G: Group](n: Subgroup[G], m: Subgroup[G], g: G, x: QuotientOver[G]) {
    is_normal_subgroup(n) and is_normal_subgroup(m) and
    m_over_n_contains(n, m, x) implies
    m_over_n_contains(n, m,
        normal_subgroup_quotient_mul(n,
            normal_subgroup_quotient_mul(n,
                normal_subgroup_quotient_project(n, g), x),
            normal_subgroup_quotient_inverse(n,
                normal_subgroup_quotient_project(n, g))))
} by {
    if is_normal_subgroup(n) and is_normal_subgroup(m) and m_over_n_contains(n, m, x) {
        let a: G satisfy {
            m.contains(a) and normal_subgroup_quotient_project(n, a) = x
        }
        normal_conjugate_mem(m, g, a)
        m.contains(g * a * g.inverse)
        normal_subgroup_quotient_project_mul(n, g, a)
        normal_subgroup_quotient_project(n, g * a) =
            normal_subgroup_quotient_mul(n,
                normal_subgroup_quotient_project(n, g),
                normal_subgroup_quotient_project(n, a))
        normal_subgroup_quotient_project_mul(n, g * a, g.inverse)
        normal_subgroup_quotient_project(n, (g * a) * g.inverse) =
            normal_subgroup_quotient_mul(n,
                normal_subgroup_quotient_project(n, g * a),
                normal_subgroup_quotient_project(n, g.inverse))
        normal_subgroup_quotient_project_inverse(n, g)
        normal_subgroup_quotient_project(n, g.inverse) =
            normal_subgroup_quotient_inverse(n, normal_subgroup_quotient_project(n, g))
        normal_subgroup_quotient_project(n, (g * a) * g.inverse) =
            normal_subgroup_quotient_mul(n,
                normal_subgroup_quotient_mul(n,
                    normal_subgroup_quotient_project(n, g),
                    normal_subgroup_quotient_project(n, a)),
                normal_subgroup_quotient_inverse(n, normal_subgroup_quotient_project(n, g)))
        normal_subgroup_quotient_project(n, (g * a) * g.inverse) =
            normal_subgroup_quotient_mul(n,
                normal_subgroup_quotient_mul(n,
                    normal_subgroup_quotient_project(n, g), x),
                normal_subgroup_quotient_inverse(n,
                    normal_subgroup_quotient_project(n, g)))
        m_over_n_contains(n, m,
            normal_subgroup_quotient_mul(n,
                normal_subgroup_quotient_mul(n,
                    normal_subgroup_quotient_project(n, g), x),
                normal_subgroup_quotient_inverse(n,
                    normal_subgroup_quotient_project(n, g))))
    }
}
