/// Functoriality of fixed-point sets for group actions.

from algebra.group import Group
from algebra.group_action import MulAction, ActionHom, is_equivariant_map,
    equivariant_map_apply, action_hom_is_equivariant,
    fixed_points, fixed_points_contains_eq, stabilizer,
    stabilizer_contains_eq
from data.basic.functions import is_injective_fn, injective_fn_eq

/// An equivariant map sends points fixed by a group element to points fixed by the same element.
theorem equivariant_map_maps_fixed_points[G: Group, X, Y](
    a: MulAction[G, X],
    b: MulAction[G, Y],
    f: X -> Y,
    g: G,
    x: X
) {
    is_equivariant_map(a, b, f) and fixed_points(a, g).contains(x) implies
        fixed_points(b, g).contains(f(x))
} by {
    if is_equivariant_map(a, b, f) and fixed_points(a, g).contains(x) {
        fixed_points_contains_eq(a, g, x)
        equivariant_map_apply(a, b, f, g, x)
        fixed_points_contains_eq(b, g, f(x))
        fixed_points(b, g).contains(f(x))
    }
}

/// An action homomorphism sends fixed points of a group element to fixed points of that element.
theorem action_hom_maps_fixed_points[G: Group, X, Y](
    f: ActionHom[G, X, Y],
    g: G,
    x: X
) {
    fixed_points(f.src, g).contains(x) implies
        fixed_points(f.dst, g).contains(f.map(x))
} by {
    if fixed_points(f.src, g).contains(x) {
        action_hom_is_equivariant(f)
        equivariant_map_maps_fixed_points(f.src, f.dst, f.map, g, x)
    }
}

/// An injective equivariant map reflects membership in fixed-point sets.
theorem equivariant_map_reflects_fixed_points_of_injective[G: Group, X, Y](
    a: MulAction[G, X],
    b: MulAction[G, Y],
    f: X -> Y,
    g: G,
    x: X
) {
    is_injective_fn(f) and is_equivariant_map(a, b, f) and
        fixed_points(b, g).contains(f(x)) implies fixed_points(a, g).contains(x)
} by {
    if is_injective_fn(f) and is_equivariant_map(a, b, f) and
        fixed_points(b, g).contains(f(x)) {
        fixed_points_contains_eq(b, g, f(x))
        equivariant_map_apply(a, b, f, g, x)
        injective_fn_eq(f, a.act(g, x), x)
        fixed_points_contains_eq(a, g, x)
        fixed_points(a, g).contains(x)
    }
}

/// An injective action homomorphism reflects membership in fixed-point sets.
theorem action_hom_reflects_fixed_points_of_injective[G: Group, X, Y](
    f: ActionHom[G, X, Y],
    g: G,
    x: X
) {
    is_injective_fn(f.map) and fixed_points(f.dst, g).contains(f.map(x)) implies
        fixed_points(f.src, g).contains(x)
} by {
    if is_injective_fn(f.map) and fixed_points(f.dst, g).contains(f.map(x)) {
        action_hom_is_equivariant(f)
        equivariant_map_reflects_fixed_points_of_injective(f.src, f.dst, f.map, g, x)
    }
}

/// Stabilizer membership of a group element is the same as membership of the point in that element's fixed-point set.
theorem stabilizer_contains_eq_fixed_points_contains[G: Group, X](
    a: MulAction[G, X],
    x: X,
    g: G
) {
    stabilizer(a, x).contains(g) = fixed_points(a, g).contains(x)
} by {
    stabilizer_contains_eq(a, x, g)
    fixed_points_contains_eq(a, g, x)
}

/// An equivariant map sends stabilizer membership in the source to fixed-point-set membership in the target.
theorem equivariant_map_maps_stabilizer_to_fixed_points[G: Group, X, Y](
    a: MulAction[G, X],
    b: MulAction[G, Y],
    f: X -> Y,
    g: G,
    x: X
) {
    is_equivariant_map(a, b, f) and stabilizer(a, x).contains(g) implies
        fixed_points(b, g).contains(f(x))
} by {
    if is_equivariant_map(a, b, f) and stabilizer(a, x).contains(g) {
        stabilizer_contains_eq_fixed_points_contains(a, x, g)
        fixed_points(a, g).contains(x)
        equivariant_map_maps_fixed_points(a, b, f, g, x)
    }
}

/// An action homomorphism sends source stabilizer membership to target fixed-point-set membership.
theorem action_hom_maps_stabilizer_to_fixed_points[G: Group, X, Y](
    f: ActionHom[G, X, Y],
    g: G,
    x: X
) {
    stabilizer(f.src, x).contains(g) implies
        fixed_points(f.dst, g).contains(f.map(x))
} by {
    if stabilizer(f.src, x).contains(g) {
        action_hom_is_equivariant(f)
        equivariant_map_maps_stabilizer_to_fixed_points(f.src, f.dst, f.map, g, x)
    }
}

/// An injective equivariant map reflects target fixed-point-set membership to source stabilizer membership.
theorem equivariant_map_reflects_fixed_points_to_stabilizer_of_injective[G: Group, X, Y](
    a: MulAction[G, X],
    b: MulAction[G, Y],
    f: X -> Y,
    g: G,
    x: X
) {
    is_injective_fn(f) and is_equivariant_map(a, b, f) and
        fixed_points(b, g).contains(f(x)) implies stabilizer(a, x).contains(g)
} by {
    if is_injective_fn(f) and is_equivariant_map(a, b, f) and
        fixed_points(b, g).contains(f(x)) {
        equivariant_map_reflects_fixed_points_of_injective(a, b, f, g, x)
        stabilizer_contains_eq_fixed_points_contains(a, x, g)
        stabilizer(a, x).contains(g)
    }
}

/// An injective action homomorphism reflects target fixed-point-set membership to source stabilizer membership.
theorem action_hom_reflects_fixed_points_to_stabilizer_of_injective[G: Group, X, Y](
    f: ActionHom[G, X, Y],
    g: G,
    x: X
) {
    is_injective_fn(f.map) and fixed_points(f.dst, g).contains(f.map(x)) implies
        stabilizer(f.src, x).contains(g)
} by {
    if is_injective_fn(f.map) and fixed_points(f.dst, g).contains(f.map(x)) {
        action_hom_is_equivariant(f)
        equivariant_map_reflects_fixed_points_to_stabilizer_of_injective(f.src, f.dst, f.map, g, x)
    }
}

/// For an injective equivariant map, fixed-point membership is equivalent before and after applying the map.
theorem equivariant_map_fixed_points_contains_eq_of_injective[G: Group, X, Y](
    a: MulAction[G, X],
    b: MulAction[G, Y],
    f: X -> Y,
    g: G,
    x: X
) {
    is_injective_fn(f) and is_equivariant_map(a, b, f) implies
        fixed_points(a, g).contains(x) = fixed_points(b, g).contains(f(x))
} by {
    if is_injective_fn(f) and is_equivariant_map(a, b, f) {
        equivariant_map_maps_fixed_points(a, b, f, g, x)
        equivariant_map_reflects_fixed_points_of_injective(a, b, f, g, x)
        if fixed_points(a, g).contains(x) {
            fixed_points(b, g).contains(f(x))
        } else {
            if fixed_points(b, g).contains(f(x)) {
                fixed_points(a, g).contains(x)
            }
            not fixed_points(b, g).contains(f(x))
        }
        fixed_points(a, g).contains(x) = fixed_points(b, g).contains(f(x))
    }
}

/// For an injective action homomorphism, fixed-point membership is equivalent before and after applying the homomorphism.
theorem action_hom_fixed_points_contains_eq_of_injective[G: Group, X, Y](
    f: ActionHom[G, X, Y],
    g: G,
    x: X
) {
    is_injective_fn(f.map) implies
        fixed_points(f.src, g).contains(x) = fixed_points(f.dst, g).contains(f.map(x))
} by {
    if is_injective_fn(f.map) {
        action_hom_is_equivariant(f)
        equivariant_map_fixed_points_contains_eq_of_injective(f.src, f.dst, f.map, g, x)
    }
}

/// For an injective equivariant map, source stabilizer membership is equivalent to target fixed-point-set membership.
theorem equivariant_map_stabilizer_fixed_points_contains_eq_of_injective[G: Group, X, Y](
    a: MulAction[G, X],
    b: MulAction[G, Y],
    f: X -> Y,
    g: G,
    x: X
) {
    is_injective_fn(f) and is_equivariant_map(a, b, f) implies
        stabilizer(a, x).contains(g) = fixed_points(b, g).contains(f(x))
} by {
    if is_injective_fn(f) and is_equivariant_map(a, b, f) {
        stabilizer_contains_eq_fixed_points_contains(a, x, g)
        equivariant_map_fixed_points_contains_eq_of_injective(a, b, f, g, x)
        stabilizer(a, x).contains(g) = fixed_points(b, g).contains(f(x))
    }
}

/// For an injective action homomorphism, source stabilizer membership is equivalent to target fixed-point-set membership.
theorem action_hom_stabilizer_fixed_points_contains_eq_of_injective[G: Group, X, Y](
    f: ActionHom[G, X, Y],
    g: G,
    x: X
) {
    is_injective_fn(f.map) implies
        stabilizer(f.src, x).contains(g) = fixed_points(f.dst, g).contains(f.map(x))
} by {
    if is_injective_fn(f.map) {
        action_hom_is_equivariant(f)
        equivariant_map_stabilizer_fixed_points_contains_eq_of_injective(f.src, f.dst, f.map, g, x)
    }
}
