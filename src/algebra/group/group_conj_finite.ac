/// Finite conjugacy-class wrappers for groups.

from algebra.group import Group
from finite_group import FiniteGroup
from algebra.group.conjugation_action import conjugation_action, conjugation_action_orbit_contains_iff
from algebra.group_action import orbit, orbit_contains_self
from finite_group import finite_orbit_list, finite_orbit_cardinality_is_length
from data.basic.set import Set, list_set, list_set_is_finite, cardinality_is_implies_is_finite
from list import List, map, map_contains_of_contains, contains_imp_unique_contains

/// The set of conjugates of `x`, expressed as its orbit under conjugation.
define conjugates_of[G: Group](x: G) -> Set[G] {
    orbit(conjugation_action[G], x)
}

/// Membership in `conjugates_of` is the usual explicit conjugacy witness.
theorem conjugates_of_contains_iff[G: Group](x: G, y: G) {
    conjugates_of(x).contains(y) iff exists(g: G) {
        g * x * g.inverse = y
    }
} by {
    conjugation_action_orbit_contains_iff(x, y)
}

/// Every element belongs to its own conjugacy class.
theorem conjugates_of_contains_self[G: Group](x: G) {
    conjugates_of(x).contains(x)
} by {
    orbit_contains_self(conjugation_action[G], x)
}

/// In a finite group, each conjugacy class is finite.
theorem conjugates_of_is_finite[G: FiniteGroup](x: G) {
    conjugates_of(x).is_finite
} by {
    finite_orbit_cardinality_is_length(conjugation_action[G], x)
    conjugates_of(x).cardinality_is(finite_orbit_list(conjugation_action[G], x).length)
    cardinality_is_implies_is_finite(conjugates_of(x),
        finite_orbit_list(conjugation_action[G], x).length)
}

/// A quotient-style wrapper for a conjugacy class.
structure ConjClasses[G: Group] {
    /// The underlying conjugacy class.
    carrier: Set[G]
} constraint {
    exists(x: G) {
        carrier = conjugates_of(x)
    }
}

/// The conjugacy-class wrapper represented by `x`.
let conj_class[G: Group](x: G) -> result: ConjClasses[G] satisfy {
    ConjClasses[G].new(conjugates_of(x)) = Option.some(result)
} by {
    exists(y: G) {
        conjugates_of(x) = conjugates_of(y)
    }
}

/// The representative wrapper has carrier `conjugates_of x`.
theorem conj_class_carrier[G: Group](x: G) {
    conj_class(x).carrier = conjugates_of(x)
} by {
    ConjClasses[G].new(conjugates_of(x)) = Option.some(conj_class(x))
}

/// Membership in the representative wrapper is explicit conjugacy.
theorem conj_class_contains_iff[G: Group](x: G, y: G) {
    conj_class(x).carrier.contains(y) iff exists(g: G) {
        g * x * g.inverse = y
    }
} by {
    conj_class_carrier(x)
    conj_class(x).carrier = conjugates_of(x)
    conjugates_of_contains_iff(x, y)
}

/// The representative belongs to its own conjugacy-class wrapper.
theorem conj_class_contains_self[G: Group](x: G) {
    conj_class(x).carrier.contains(x)
} by {
    conj_class_carrier(x)
    conjugates_of_contains_self(x)
}

/// The representative wrapper has finite carrier in a finite group.
theorem conj_class_rep_carrier_is_finite[G: FiniteGroup](x: G) {
    conj_class(x).carrier.is_finite
} by {
    conj_class_carrier(x)
    conjugates_of_is_finite(x)
}

/// A conjugacy-class wrapper in a finite group has finite carrier.
theorem conj_class_carrier_is_finite[G: FiniteGroup](c: ConjClasses[G]) {
    c.carrier.is_finite
} by {
    let x: G satisfy {
        c.carrier = conjugates_of(x)
    }
    conjugates_of_is_finite(x)
    c.carrier.is_finite
}

/// A finite list of all conjugacy-class quotient wrappers of a finite group.
let finite_conj_classes_list[G: FiniteGroup]: List[ConjClasses[G]] =
    map[G, ConjClasses[G]](G.elements, conj_class[G]).unique

/// The finite set of conjugacy-class quotient wrappers of a finite group.
let finite_conj_classes[G: FiniteGroup]: Set[ConjClasses[G]] =
    list_set(finite_conj_classes_list[G])

/// A represented conjugacy class is listed in the finite conjugacy-class list.
theorem finite_conj_classes_list_contains_conj_class[G: FiniteGroup](x: G) {
    finite_conj_classes_list[G].contains(conj_class(x))
} by {
    G.elements.contains_every
    G.elements.contains(x)
    map_contains_of_contains(G.elements, conj_class[G], x)
    map[G, ConjClasses[G]](G.elements, conj_class[G]).contains(conj_class(x))
    contains_imp_unique_contains(map[G, ConjClasses[G]](G.elements, conj_class[G]), conj_class(x))
    finite_conj_classes_list[G].contains(conj_class(x))
}

/// A represented conjugacy class belongs to the finite conjugacy-class set.
theorem finite_conj_classes_contains_conj_class[G: FiniteGroup](x: G) {
    finite_conj_classes[G].contains(conj_class(x))
} by {
    finite_conj_classes_list_contains_conj_class(x)
    finite_conj_classes[G] = list_set(finite_conj_classes_list[G])
    finite_conj_classes[G].contains(conj_class(x)) =
        finite_conj_classes_list[G].contains(conj_class(x))
    finite_conj_classes[G].contains(conj_class(x))
}

/// Finite groups have finitely many conjugacy-class wrappers.
theorem finite_conj_classes_is_finite[G: FiniteGroup] {
    finite_conj_classes[G].is_finite
} by {
    list_set_is_finite(finite_conj_classes_list[G])
}
