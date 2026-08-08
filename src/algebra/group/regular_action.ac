/// Regular actions of a group on itself.

from algebra.group import Group, inverse_left, inverse_inverse, inverse_mul, left_cancel, right_cancel
from algebra.group_action import MulAction, action_identity_constraint, action_mul_constraint,
    is_mul_action, mul_action_one, mul_action_mul, orbit, orbit_contains_action,
    orbit_contains_witness, is_transitive_action, is_faithful_action, stabilizer,
    stabilizer_contains_of_fixes, stabilizer_fixes

/// The left regular action map of a group on itself.
define left_regular_act[G: Group](g: G, x: G) -> G {
    g * x
}

/// The left regular action map satisfies the identity constraint.
theorem left_regular_act_identity_constraint[G: Group] {
    action_identity_constraint(left_regular_act[G])
} by {
    forall(x: G) {
        left_regular_act[G](G.1, x) = G.1 * x
        G.1 * x = x
        left_regular_act[G](G.1, x) = x
    }
}

/// The left regular action map satisfies the multiplication constraint.
theorem left_regular_act_mul_constraint[G: Group] {
    action_mul_constraint(left_regular_act[G])
} by {
    forall(g: G, h: G, x: G) {
        left_regular_act[G](g * h, x) = (g * h) * x
        (g * h) * x = g * (h * x)
        left_regular_act[G](h, x) = h * x
        left_regular_act[G](g, left_regular_act[G](h, x)) = g * (h * x)
        left_regular_act[G](g * h, x) = left_regular_act[G](g, left_regular_act[G](h, x))
    }
}

/// The left regular action map is a group action.
theorem left_regular_act_is_mul_action[G: Group] {
    is_mul_action(left_regular_act[G])
} by {
    left_regular_act_mul_constraint[G]
    left_regular_act_identity_constraint[G]
    is_mul_action(left_regular_act[G])
}

/// The bundled left regular action of a group on itself.
let left_regular_action[G: Group]: MulAction[G, G] satisfy {
    MulAction.new(left_regular_act[G]) = Option.some(left_regular_action)
}

/// The left regular action acts by left multiplication.
theorem left_regular_action_act[G: Group](g: G, x: G) {
    left_regular_action[G].act(g, x) = g * x
} by {
    left_regular_action[G].act(g, x) = left_regular_act[G](g, x)
    left_regular_act[G](g, x) = g * x
}

/// Any target lies in the left regular orbit of any source.
theorem left_regular_action_orbit_contains[G: Group](x: G, y: G) {
    orbit(left_regular_action[G], x).contains(y)
} by {
    let g = y * x.inverse
    inverse_left(x)
    x.inverse * x = G.1
    g * x = (y * x.inverse) * x
    (y * x.inverse) * x = y * (x.inverse * x)
    y * (x.inverse * x) = y * G.1
    y * G.1 = y
    left_regular_action_act(g, x)
    left_regular_action[G].act(g, x) = g * x
    left_regular_action[G].act(g, x) = y
    orbit_contains_action(left_regular_action[G], x, g)
    orbit(left_regular_action[G], x).contains(left_regular_action[G].act(g, x))
    orbit(left_regular_action[G], x).contains(y)
}

/// The left regular action is transitive.
theorem left_regular_action_is_transitive[G: Group] {
    is_transitive_action(left_regular_action[G])
} by {
    forall(x: G, y: G) {
        left_regular_action_orbit_contains(x, y)
        orbit(left_regular_action[G], x).contains(y)
    }
    is_transitive_action(left_regular_action[G])
}

/// Equal left translations are equal group elements.
theorem left_regular_action_eq_of_pointwise_eq[G: Group](g: G, h: G) {
    (forall(x: G) { left_regular_action[G].act(g, x) = left_regular_action[G].act(h, x) }) implies
        g = h
} by {
    if forall(x: G) { left_regular_action[G].act(g, x) = left_regular_action[G].act(h, x) } {
        left_regular_action_act(g, G.1)
        left_regular_action_act(h, G.1)
        left_regular_action[G].act(g, G.1) = g * G.1
        left_regular_action[G].act(h, G.1) = h * G.1
        g * G.1 = h * G.1
        g * G.1 = g
        h * G.1 = h
        g = h
    }
}

/// The left regular action is faithful.
theorem left_regular_action_is_faithful[G: Group] {
    is_faithful_action(left_regular_action[G])
} by {
    forall(g: G, h: G) {
        if forall(x: G) { left_regular_action[G].act(g, x) = left_regular_action[G].act(h, x) } {
            left_regular_action_eq_of_pointwise_eq(g, h)
            g = h
        }
    }
}

/// Fixing one point in the left regular action means being the identity.
theorem left_regular_action_fix_eq_one[G: Group](g: G, x: G) {
    left_regular_action[G].act(g, x) = x implies g = G.1
} by {
    if left_regular_action[G].act(g, x) = x {
        left_regular_action_act(g, x)
        g * x = x
        G.1 * x = x
        g * x = G.1 * x
        right_cancel(x, g, G.1)
        g = G.1
    }
}

/// The identity fixes every point in the left regular action.
theorem left_regular_action_one_fixes[G: Group](x: G) {
    left_regular_action[G].act(G.1, x) = x
} by {
    mul_action_one(left_regular_action[G], x)
}

/// Stabilizer membership for the left regular action is equality with the identity.
theorem left_regular_action_stabilizer_contains_iff[G: Group](x: G, g: G) {
    stabilizer(left_regular_action[G], x).contains(g) iff g = G.1
} by {
    if stabilizer(left_regular_action[G], x).contains(g) {
        stabilizer_fixes(left_regular_action[G], x, g)
        left_regular_action_fix_eq_one(g, x)
        g = G.1
    }
    if g = G.1 {
        left_regular_action_one_fixes(x)
        left_regular_action[G].act(g, x) = x
        stabilizer_contains_of_fixes(left_regular_action[G], x, g)
        stabilizer(left_regular_action[G], x).contains(g)
    }
}

/// Stabilizers in the left regular action are trivial.
theorem left_regular_action_stabilizer_trivial[G: Group](x: G, g: G) {
    stabilizer(left_regular_action[G], x).contains(g) implies g = G.1
} by {
    if stabilizer(left_regular_action[G], x).contains(g) {
        left_regular_action_stabilizer_contains_iff(x, g)
        g = G.1
    }
}

/// The right regular action map of a group on itself.
define right_regular_act[G: Group](g: G, x: G) -> G {
    x * g.inverse
}

/// The inverse of the identity element is the identity element.
theorem one_inverse_eq_one[G: Group] {
    G.1.inverse = G.1
} by {
    G.1.inverse * G.1 = G.1.inverse
    inverse_left(G.1)
    G.1.inverse * G.1 = G.1
    G.1.inverse = G.1
}

/// The right regular action map satisfies the identity constraint.
theorem right_regular_act_identity_constraint[G: Group] {
    action_identity_constraint(right_regular_act[G])
} by {
    forall(x: G) {
        one_inverse_eq_one[G]
        G.1.inverse = G.1
        right_regular_act[G](G.1, x) = x * G.1.inverse
        x * G.1.inverse = x * G.1
        x * G.1 = x
        right_regular_act[G](G.1, x) = x
    }
}

/// The inverse of a product is ordered for the right regular action.
theorem right_regular_product_inverse[G: Group](g: G, h: G) {
    (g * h).inverse = h.inverse * g.inverse
} by {
    inverse_mul(g, h)
}

/// The right regular action map satisfies the multiplication constraint.
theorem right_regular_act_mul_constraint[G: Group] {
    action_mul_constraint(right_regular_act[G])
} by {
    forall(g: G, h: G, x: G) {
        right_regular_product_inverse(g, h)
        (g * h).inverse = h.inverse * g.inverse
        right_regular_act[G](g * h, x) = x * (g * h).inverse
        right_regular_act[G](g * h, x) = x * (h.inverse * g.inverse)
        x * (h.inverse * g.inverse) = (x * h.inverse) * g.inverse
        right_regular_act[G](h, x) = x * h.inverse
        right_regular_act[G](g, right_regular_act[G](h, x)) =
            right_regular_act[G](h, x) * g.inverse
        right_regular_act[G](g, right_regular_act[G](h, x)) =
            (x * h.inverse) * g.inverse
        right_regular_act[G](g * h, x) = right_regular_act[G](g, right_regular_act[G](h, x))
    }
}

/// The right regular action map is a group action.
theorem right_regular_act_is_mul_action[G: Group] {
    is_mul_action(right_regular_act[G])
} by {
    right_regular_act_mul_constraint[G]
    right_regular_act_identity_constraint[G]
    is_mul_action(right_regular_act[G])
}

/// The bundled right regular action of a group on itself.
let right_regular_action[G: Group]: MulAction[G, G] satisfy {
    MulAction.new(right_regular_act[G]) = Option.some(right_regular_action)
}

/// The right regular action acts by right multiplication by the inverse.
theorem right_regular_action_act[G: Group](g: G, x: G) {
    right_regular_action[G].act(g, x) = x * g.inverse
} by {
    right_regular_action[G].act(g, x) = right_regular_act[G](g, x)
    right_regular_act[G](g, x) = x * g.inverse
}

/// The inverse of the element sending `x` to `y` in the right regular action.
theorem right_regular_witness_inverse[G: Group](x: G, y: G) {
    (y.inverse * x).inverse = x.inverse * y
} by {
    inverse_mul(y.inverse, x)
    (y.inverse * x).inverse = x.inverse * y.inverse.inverse
    inverse_inverse(y)
    y.inverse.inverse = y
    (y.inverse * x).inverse = x.inverse * y
}

/// Any target lies in the right regular orbit of any source.
theorem right_regular_action_orbit_contains[G: Group](x: G, y: G) {
    orbit(right_regular_action[G], x).contains(y)
} by {
    let g = y.inverse * x
    right_regular_witness_inverse(x, y)
    g.inverse = x.inverse * y
    right_regular_action_act(g, x)
    right_regular_action[G].act(g, x) = x * g.inverse
    x * g.inverse = x * (x.inverse * y)
    x * (x.inverse * y) = (x * x.inverse) * y
    x * x.inverse = G.1
    (x * x.inverse) * y = G.1 * y
    G.1 * y = y
    right_regular_action[G].act(g, x) = y
    orbit_contains_action(right_regular_action[G], x, g)
    orbit(right_regular_action[G], x).contains(right_regular_action[G].act(g, x))
    orbit(right_regular_action[G], x).contains(y)
}

/// The right regular action is transitive.
theorem right_regular_action_is_transitive[G: Group] {
    is_transitive_action(right_regular_action[G])
} by {
    forall(x: G, y: G) {
        right_regular_action_orbit_contains(x, y)
        orbit(right_regular_action[G], x).contains(y)
    }
    is_transitive_action(right_regular_action[G])
}

/// Equal right translations are equal group elements.
theorem right_regular_action_eq_of_pointwise_eq[G: Group](g: G, h: G) {
    (forall(x: G) { right_regular_action[G].act(g, x) = right_regular_action[G].act(h, x) }) implies
        g = h
} by {
    if forall(x: G) { right_regular_action[G].act(g, x) = right_regular_action[G].act(h, x) } {
        right_regular_action_act(g, G.1)
        right_regular_action_act(h, G.1)
        right_regular_action[G].act(g, G.1) = G.1 * g.inverse
        right_regular_action[G].act(h, G.1) = G.1 * h.inverse
        G.1 * g.inverse = G.1 * h.inverse
        G.1 * g.inverse = g.inverse
        G.1 * h.inverse = h.inverse
        g.inverse = h.inverse
        inverse_inverse(g)
        inverse_inverse(h)
        g.inverse.inverse = h.inverse.inverse
        g = h
    }
}

/// The right regular action is faithful.
theorem right_regular_action_is_faithful[G: Group] {
    is_faithful_action(right_regular_action[G])
} by {
    forall(g: G, h: G) {
        if forall(x: G) { right_regular_action[G].act(g, x) = right_regular_action[G].act(h, x) } {
            right_regular_action_eq_of_pointwise_eq(g, h)
            g = h
        }
    }
}

/// Fixing one point in the right regular action means being the identity.
theorem right_regular_action_fix_eq_one[G: Group](g: G, x: G) {
    right_regular_action[G].act(g, x) = x implies g = G.1
} by {
    if right_regular_action[G].act(g, x) = x {
        right_regular_action_act(g, x)
        x * g.inverse = x
        x * G.1 = x
        x * g.inverse = x * G.1
        left_cancel(x, g.inverse, G.1)
        g.inverse = G.1
        G.1.inverse = g.inverse.inverse
        one_inverse_eq_one[G]
        G.1.inverse = G.1
        inverse_inverse(g)
        g.inverse.inverse = g
        G.1 = g
        g = G.1
    }
}

/// The identity fixes every point in the right regular action.
theorem right_regular_action_one_fixes[G: Group](x: G) {
    right_regular_action[G].act(G.1, x) = x
} by {
    mul_action_one(right_regular_action[G], x)
}

/// Stabilizer membership for the right regular action is equality with the identity.
theorem right_regular_action_stabilizer_contains_iff[G: Group](x: G, g: G) {
    stabilizer(right_regular_action[G], x).contains(g) iff g = G.1
} by {
    if stabilizer(right_regular_action[G], x).contains(g) {
        stabilizer_fixes(right_regular_action[G], x, g)
        right_regular_action_fix_eq_one(g, x)
        g = G.1
    }
    if g = G.1 {
        right_regular_action_one_fixes(x)
        right_regular_action[G].act(g, x) = x
        stabilizer_contains_of_fixes(right_regular_action[G], x, g)
        stabilizer(right_regular_action[G], x).contains(g)
    }
}

/// Stabilizers in the right regular action are trivial.
theorem right_regular_action_stabilizer_trivial[G: Group](x: G, g: G) {
    stabilizer(right_regular_action[G], x).contains(g) implies g = G.1
} by {
    if stabilizer(right_regular_action[G], x).contains(g) {
        right_regular_action_stabilizer_contains_iff(x, g)
        g = G.1
    }
}
