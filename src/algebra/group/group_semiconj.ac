/// Semiconjugate elements of a semigroup.

from algebra.group import Group, inverse_left
from algebra.mul import Mul
from algebra.monoid.monoid import Monoid
from algebra.semigroup import Semigroup

/// True if `x` is semiconjugate to `y` by `a`.
define semiconj_by[M: Mul](a: M, x: M, y: M) -> Bool {
    a * x = y * a
}

/// The definition of semiconjugacy is its underlying multiplication equality.
theorem semiconj_by_eq[M: Mul](a: M, x: M, y: M) {
    semiconj_by(a, x, y) = (a * x = y * a)
} by {
    semiconj_by(a, x, y) = (a * x = y * a)
}

/// Semiconjugacy gives the underlying multiplication equality.
theorem semiconj_by_mul_eq[M: Mul](a: M, x: M, y: M) {
    semiconj_by(a, x, y) implies a * x = y * a
} by {
    if semiconj_by(a, x, y) {
        semiconj_by_eq(a, x, y)
        a * x = y * a
    }
}

/// One element semiconjugates a product on the right when it semiconjugates each factor.
theorem semiconj_by_mul_right[S: Semigroup](a: S, x: S, y: S, xp: S, yp: S) {
    semiconj_by(a, x, y) and semiconj_by(a, xp, yp)
    implies semiconj_by(a, x * xp, y * yp)
} by {
    if semiconj_by(a, x, y) and semiconj_by(a, xp, yp) {
        semiconj_by_mul_eq(a, x, y)
        semiconj_by_mul_eq(a, xp, yp)
        a * x = y * a
        a * xp = yp * a
        a * (x * xp) = (a * x) * xp
        (a * x) * xp = (y * a) * xp
        y * (a * xp) = (y * a) * xp
        (y * a) * xp = y * (a * xp)
        y * (a * xp) = y * (yp * a)
        y * (yp * a) = (y * yp) * a
        a * (x * xp) = (y * yp) * a
        semiconj_by(a, x * xp, y * yp)
    }
}

/// Products on the left compose semiconjugacies.
theorem semiconj_by_mul_left[S: Semigroup](a: S, b: S, x: S, y: S, z: S) {
    semiconj_by(a, y, z) and semiconj_by(b, x, y)
    implies semiconj_by(a * b, x, z)
} by {
    if semiconj_by(a, y, z) and semiconj_by(b, x, y) {
        semiconj_by_mul_eq(a, y, z)
        semiconj_by_mul_eq(b, x, y)
        a * y = z * a
        b * x = y * b
        a * (b * x) = (a * b) * x
        (a * b) * x = a * (b * x)
        a * (b * x) = a * (y * b)
        a * (y * b) = (a * y) * b
        (a * y) * b = (z * a) * b
        z * (a * b) = (z * a) * b
        (z * a) * b = z * (a * b)
        (a * b) * x = z * (a * b)
        semiconj_by(a * b, x, z)
    }
}

/// Any element semiconjugates the identity element to itself.
theorem semiconj_by_one_right[M: Monoid](a: M) {
    semiconj_by(a, M.1, M.1)
} by {
    a * M.1 = a
    M.1 * a = a
    a * M.1 = M.1 * a
}

/// The identity element semiconjugates any element to itself.
theorem semiconj_by_one_left[M: Monoid](x: M) {
    semiconj_by(M.1, x, x)
} by {
    M.1 * x = x
    x * M.1 = x
    M.1 * x = x * M.1
}

/// An element semiconjugates `x` to its conjugate by that element.
theorem semiconj_by_conj_mk[G: Group](a: G, x: G) {
    semiconj_by(a, x, a * x * a.inverse)
} by {
    inverse_left(a)
    a.inverse * a = G.1
    (a * x * a.inverse) * a = (a * x) * (a.inverse * a)
    (a * x) * (a.inverse * a) = (a * x) * G.1
    (a * x) * G.1 = a * x
    (a * x * a.inverse) * a = a * x
    a * x = (a * x * a.inverse) * a
}
