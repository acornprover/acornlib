/// The first isomorphism theorem for groups, packaged: the induced map from
/// the kernel quotient to the codomain is injective, and its image is exactly
/// the image of the homomorphism.  The universal machinery lives in
/// `data.basic.quotient_algebra` and `data.basic.quotient_algebra_first_iso`;
/// this file restates it in the bundled vocabulary.

from algebra.group import Group, GroupHom
from data.basic.equivalence import QuotientOver
from data.basic.quotient_algebra import group_hom_kernel_quotient_induced,
    group_hom_kernel_quotient_project, group_hom_kernel_quotient_induced_injective,
    group_hom_kernel_quotient_induced_surjective_onto_image
from data.basic.quotient_algebra_first_iso import group_hom_kernel_quotient_image_eq_iff_project_eq
from nat import Nat

numerals Nat

/// The kernel-quotient induced map of a group homomorphism is injective.
theorem first_iso_group_induced_injective[G: Group, H: Group](
    f: GroupHom[G, H], a: G, b: G
) {
    group_hom_kernel_quotient_induced(f, group_hom_kernel_quotient_project(f, a)) =
        group_hom_kernel_quotient_induced(f, group_hom_kernel_quotient_project(f, b))
    implies group_hom_kernel_quotient_project(f, a) = group_hom_kernel_quotient_project(f, b)
} by {
    group_hom_kernel_quotient_induced_injective(f, a, b)
}

/// The kernel-quotient induced map hits every element of the homomorphism image.
theorem first_iso_group_induced_onto_image[G: Group, H: Group](f: GroupHom[G, H], a: G) {
    exists(q: QuotientOver[G]) {
        group_hom_kernel_quotient_induced(f, q) = f.hom(a)
    }
} by {
    group_hom_kernel_quotient_induced_surjective_onto_image(f, a)
}

/// Projections through the kernel quotient agree exactly when the images agree:
/// the kernel-quotient fiber of a value is the fiber of that value under the
/// homomorphism.
theorem first_iso_group_project_eq_iff_image_eq[G: Group, H: Group](f: GroupHom[G, H], a: G, b: G) {
    (group_hom_kernel_quotient_project(f, a) = group_hom_kernel_quotient_project(f, b)) =
        (f.hom(a) = f.hom(b))
} by {
    group_hom_kernel_quotient_image_eq_iff_project_eq(f, a, b)
}
