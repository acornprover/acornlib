from algebra.group import Group, inverse_inverse, inverse_mul, inverse_left, left_cancel
from nat import pow_add, pow_pow, pow_zero
from algebra.comm_group import CommGroup
from algebra.subgroup import Subgroup, subgroup_contains_identity, subgroup_mul_mem, subgroup_inv_mem,
    subgroup_contains_inverse_iff
from data.basic.relation_basic import is_reflexive, is_symmetric, is_transitive, is_equivalence
from data.basic.relation_transport import respects_mul, is_mul_congruence, respects_binary_op,
    respects_unary_op, is_unary_congruence, is_binary_congruence, respects_mul_eq_respects_binary_op,
    mul_congruence_eq_binary_congruence
from data.basic.quotient_algebra import is_group_congruence
from data.basic.equivalence import QuotientRelation, quotient_relation_new_round_trip,
    quotient_binary_respects, quotient_binary_respects_eq_respects_binary_op,
    QuotientOver, quotient_over_mk, quotient_unary_respects,
    quotient_unary_respects_eq_respects_unary_op,
    quotient_over_binary_op, quotient_over_binary_op_projection,
    quotient_over_binary_op_projection_left, quotient_over_binary_op_projection_right,
    quotient_over_unary_op, quotient_over_unary_op_projection,
    quotient_over_unary_op_projection_compatible, quotient_over_mk_eq_of_rel,
    rel_of_quotient_over_mk_eq, quotient_over_mk_eq_iff_rel
from nat import Nat, alt_induction

/// True if a subgroup is normal: it is invariant under conjugation.
define is_normal_subgroup[G: Group](s: Subgroup[G]) -> Bool {
    forall(g: G, n: G) {
        s.contains(n) implies s.contains(g * n * g.inverse)
    }
}

/// Normality applied to a specific conjugating element.
theorem normal_conjugate_mem[G: Group](s: Subgroup[G], g: G, n: G) {
    is_normal_subgroup(s) and s.contains(n) implies s.contains(g * n * g.inverse)
} by {
    if is_normal_subgroup(s) and s.contains(n) {
        is_normal_subgroup(s) = forall(x: G, y: G) {
            s.contains(y) implies s.contains(x * y * x.inverse)
        }
        s.contains(g * n * g.inverse)
    }
}

/// The equivalence relation on `G` defined by a subgroup: `a ~ b` iff `a * b.inverse` lies in the subgroup.
define normal_subgroup_rel[G: Group](s: Subgroup[G], a: G, b: G) -> Bool {
    s.contains(a * b.inverse)
}

/// The relation defined by a subgroup is reflexive.
theorem normal_subgroup_rel_reflexive[G: Group](s: Subgroup[G], a: G) {
    normal_subgroup_rel(s, a, a)
} by {
    subgroup_contains_identity(s)
}

/// The relation defined by a subgroup is symmetric.
theorem normal_subgroup_rel_symmetric[G: Group](s: Subgroup[G], a: G, b: G) {
    normal_subgroup_rel(s, a, b) implies normal_subgroup_rel(s, b, a)
} by {
    if normal_subgroup_rel(s, a, b) {
        subgroup_inv_mem(s, a * b.inverse)
        inverse_mul(a, b.inverse)
        inverse_inverse(b)
        s.contains(b * a.inverse)
    }
}

/// Inverse multiplication chains through an intermediate point.
theorem normal_subgroup_inv_chain[G: Group](a: G, b: G, c: G) {
    (a * b.inverse) * (b * c.inverse) = a * c.inverse
} by {
    (a * b.inverse) * (b * c.inverse) = a * (b.inverse * b) * c.inverse
}

/// Pointwise transitivity helper for the subgroup relation.
theorem normal_subgroup_rel_trans_at[G: Group](s: Subgroup[G], a: G, b: G, c: G) {
    normal_subgroup_rel(s, a, b) and normal_subgroup_rel(s, b, c)
    implies normal_subgroup_rel(s, a, c)
} by {
    if normal_subgroup_rel(s, a, b) and normal_subgroup_rel(s, b, c) {
        s.contains(b * c.inverse)
        subgroup_mul_mem(s, a * b.inverse, b * c.inverse)
        s.contains((a * b.inverse) * (b * c.inverse))
        normal_subgroup_inv_chain(a, b, c)
        s.contains(a * c.inverse)
    }
}

/// The relation defined by a subgroup is reflexive.
theorem normal_subgroup_rel_is_reflexive[G: Group](s: Subgroup[G]) {
    is_reflexive(normal_subgroup_rel(s))
} by {
    forall(a: G) {
        normal_subgroup_rel_reflexive(s, a)
    }
}

/// The relation defined by a subgroup is symmetric.
theorem normal_subgroup_rel_is_symmetric[G: Group](s: Subgroup[G]) {
    is_symmetric(normal_subgroup_rel(s))
} by {
    forall(a: G, b: G) {
        if normal_subgroup_rel(s, a, b) {
            normal_subgroup_rel_symmetric(s, a, b)
        }
    }
}

/// The relation defined by a subgroup composes consecutive related pairs.
theorem normal_subgroup_rel_transitive_statement[G: Group](s: Subgroup[G]) {
    forall(a: G, b: G, c: G) {
        normal_subgroup_rel(s, a, b) and normal_subgroup_rel(s, b, c)
        implies normal_subgroup_rel(s, a, c)
    }
} by {
    forall(a: G, b: G, c: G) {
        if normal_subgroup_rel(s, a, b) and normal_subgroup_rel(s, b, c) {
            normal_subgroup_rel_trans_at(s, a, b, c)
            normal_subgroup_rel(s, a, c)
        }
    }
}

/// The relation defined by a subgroup is transitive.
theorem normal_subgroup_rel_is_transitive[G: Group](s: Subgroup[G]) {
    is_transitive(normal_subgroup_rel(s))
} by {
    let r: (G, G) -> Bool = normal_subgroup_rel(s)
    forall(a: G, b: G, c: G) {
        if r(a, b) and r(b, c) {
            r(b, c) = normal_subgroup_rel(s, b, c)
            normal_subgroup_rel_trans_at(s, a, b, c)
            r(a, c)
        }
    }
    is_transitive(r) = forall(x: G, y: G, z: G) {
        r(x, y) and r(y, z) implies r(x, z)
    }
    is_transitive(r)
}

/// The product of `a * b` and the inverse of `ap * bp` regroups as a conjugate of `b * bp.inverse` times `a * ap.inverse`.
theorem normal_mul_inverse_distrib[G: Group](a: G, b: G, ap: G, bp: G) {
    (a * b) * (ap * bp).inverse = (a * (b * bp.inverse) * a.inverse) * (a * ap.inverse)
} by {
    inverse_mul(ap, bp)
    a * b * bp.inverse * ap.inverse = a * (b * bp.inverse) * (a.inverse * a) * ap.inverse
}

/// Pointwise multiplicative-step helper: the normal-subgroup relation is preserved by pairwise multiplication.
theorem normal_subgroup_rel_mul_step[G: Group](s: Subgroup[G],
        a: G, ap: G, b: G, bp: G) {
    is_normal_subgroup(s) and normal_subgroup_rel(s, a, ap) and normal_subgroup_rel(s, b, bp)
    implies normal_subgroup_rel(s, a * b, ap * bp)
} by {
    if is_normal_subgroup(s) and normal_subgroup_rel(s, a, ap) and normal_subgroup_rel(s, b, bp) {
        normal_conjugate_mem(s, a, b * bp.inverse)
        s.contains(a * (b * bp.inverse) * a.inverse)
        subgroup_mul_mem(s, a * (b * bp.inverse) * a.inverse, a * ap.inverse)
        s.contains((a * (b * bp.inverse) * a.inverse) * (a * ap.inverse))
        normal_mul_inverse_distrib(a, b, ap, bp)
        s.contains((a * b) * (ap * bp).inverse)
    }
}

/// Pointwise inverse-step helper: the normal-subgroup relation is preserved by inversion.
theorem normal_subgroup_rel_inv_step[G: Group](s: Subgroup[G], a: G, b: G) {
    is_normal_subgroup(s) and normal_subgroup_rel(s, a, b)
    implies normal_subgroup_rel(s, a.inverse, b.inverse)
} by {
    if is_normal_subgroup(s) and normal_subgroup_rel(s, a, b) {
        normal_subgroup_rel_symmetric(s, a, b)
        normal_conjugate_mem(s, b.inverse, b * a.inverse)
        s.contains(b.inverse * (b * a.inverse) * b.inverse.inverse)
        inverse_inverse(b)
        (b.inverse * b) * a.inverse * b = a.inverse * b
        s.contains(a.inverse * b.inverse.inverse)
    }
}


/// The normal-subgroup relation is compatible with multiplication when the subgroup is normal.
theorem normal_subgroup_rel_respects_mul[G: Group](s: Subgroup[G]) {
    is_normal_subgroup(s) implies respects_mul(normal_subgroup_rel(s))
} by {
    if is_normal_subgroup(s) {
        respects_mul_eq_respects_binary_op(normal_subgroup_rel(s))
        forall(a: G, ap: G, b: G, bp: G) {
            if normal_subgroup_rel(s, a, ap) and normal_subgroup_rel(s, b, bp) {
                normal_subgroup_rel_mul_step(s, a, ap, b, bp)
                normal_subgroup_rel(s, G.mul(a, b), G.mul(ap, bp))
            }
        }
        respects_binary_op(normal_subgroup_rel(s), G.mul)
    }
}

/// The normal-subgroup relation is compatible with inversion when the subgroup is normal.
theorem normal_subgroup_rel_respects_inverse[G: Group](s: Subgroup[G]) {
    is_normal_subgroup(s) implies respects_unary_op(normal_subgroup_rel(s), G.inverse)
} by {
    if is_normal_subgroup(s) {
        forall(a: G, b: G) {
            if normal_subgroup_rel(s, a, b) {
                normal_subgroup_rel_inv_step(s, a, b)
                normal_subgroup_rel(s, G.inverse(a), G.inverse(b))
            }
        }
    }
}

/// The relation defined by a subgroup is an equivalence relation.
theorem normal_subgroup_rel_is_equivalence[G: Group](s: Subgroup[G]) {
    is_equivalence(normal_subgroup_rel(s))
} by {
    normal_subgroup_rel_is_reflexive(s)
    normal_subgroup_rel_is_symmetric(s)
    normal_subgroup_rel_is_transitive(s)
}

/// The quotient relation associated with a subgroup.
let normal_subgroup_quotient_relation[G: Group](s: Subgroup[G]) -> result: QuotientRelation[G] satisfy {
    QuotientRelation[G].new(normal_subgroup_rel(s)) = Option.some(result)
} by {
    normal_subgroup_rel_is_equivalence(s)
}

/// The quotient relation associated with a subgroup has the expected underlying relation.
theorem normal_subgroup_quotient_relation_rel[G: Group](s: Subgroup[G]) {
    normal_subgroup_quotient_relation(s).rel = normal_subgroup_rel(s)
} by {
    quotient_relation_new_round_trip(normal_subgroup_rel(s), normal_subgroup_quotient_relation(s))
}

/// The normal-subgroup relation is a multiplicative congruence when the subgroup is normal.
theorem normal_subgroup_rel_is_mul_congruence[G: Group](s: Subgroup[G]) {
    is_normal_subgroup(s) implies is_mul_congruence(normal_subgroup_rel(s))
} by {
    if is_normal_subgroup(s) {
        normal_subgroup_rel_is_equivalence(s)
        normal_subgroup_rel_respects_mul(s)
        respects_mul_eq_respects_binary_op(normal_subgroup_rel(s))
        is_binary_congruence(normal_subgroup_rel(s), G.mul)
        mul_congruence_eq_binary_congruence(normal_subgroup_rel(s))
    }
}

/// The normal-subgroup relation is a group congruence when the subgroup is normal.
theorem normal_subgroup_rel_is_group_congruence[G: Group](s: Subgroup[G]) {
    is_normal_subgroup(s) implies is_group_congruence(normal_subgroup_rel(s))
} by {
    if is_normal_subgroup(s) {
        normal_subgroup_rel_is_mul_congruence(s)
        normal_subgroup_rel_is_equivalence(s)
        normal_subgroup_rel_respects_inverse(s)
        is_unary_congruence(normal_subgroup_rel(s), G.inverse)
    }
}

/// The quotient relation built from a normal subgroup respects multiplication as a quotient binary operation.
theorem normal_subgroup_quotient_relation_respects_mul[G: Group](s: Subgroup[G]) {
    is_normal_subgroup(s) implies quotient_binary_respects(normal_subgroup_quotient_relation(s), G.mul)
} by {
    if is_normal_subgroup(s) {
        quotient_binary_respects_eq_respects_binary_op(normal_subgroup_quotient_relation(s), G.mul)
        normal_subgroup_rel_respects_mul(s)
        respects_mul_eq_respects_binary_op(normal_subgroup_rel(s))
        normal_subgroup_quotient_relation_rel(s)
        respects_binary_op(normal_subgroup_quotient_relation(s).rel, G.mul)
    }
}

/// The quotient relation built from a normal subgroup respects inversion as a quotient unary operation.
theorem normal_subgroup_quotient_relation_respects_inverse[G: Group](s: Subgroup[G]) {
    is_normal_subgroup(s) implies quotient_unary_respects(normal_subgroup_quotient_relation(s), G.inverse)
} by {
    if is_normal_subgroup(s) {
        quotient_unary_respects_eq_respects_unary_op(normal_subgroup_quotient_relation(s), G.inverse)
        normal_subgroup_rel_respects_inverse(s)
        normal_subgroup_quotient_relation_rel(s)
        respects_unary_op(normal_subgroup_quotient_relation(s).rel, G.inverse)
    }
}

/// Multiplication on the relation-indexed quotient by a normal subgroup.
define normal_subgroup_quotient_mul[G: Group](s: Subgroup[G]) ->
    (QuotientOver[G], QuotientOver[G]) -> QuotientOver[G] {
    quotient_over_binary_op(G.mul)
}

/// Multiplication in the normal-subgroup quotient agrees with multiplication of representatives.
theorem normal_subgroup_quotient_mul_projection[G: Group](s: Subgroup[G], a: G, b: G) {
    is_normal_subgroup(s) implies normal_subgroup_quotient_mul(s,
        quotient_over_mk(normal_subgroup_quotient_relation(s), a),
        quotient_over_mk(normal_subgroup_quotient_relation(s), b)
    ) = quotient_over_mk(normal_subgroup_quotient_relation(s), a * b)
} by {
    if is_normal_subgroup(s) {
        normal_subgroup_quotient_relation_respects_mul(s)
        quotient_over_binary_op_projection(normal_subgroup_quotient_relation(s), G.mul, a, b)
        G.mul(a, b) = a * b
    }
}

/// Multiplication in the normal-subgroup quotient respects related left representatives.
theorem normal_subgroup_quotient_mul_projection_left[G: Group](s: Subgroup[G], a1: G, a2: G, b: G) {
    is_normal_subgroup(s) and normal_subgroup_rel(s, a1, a2) implies
    normal_subgroup_quotient_mul(s,
        quotient_over_mk(normal_subgroup_quotient_relation(s), a1),
        quotient_over_mk(normal_subgroup_quotient_relation(s), b)
    ) = normal_subgroup_quotient_mul(s,
        quotient_over_mk(normal_subgroup_quotient_relation(s), a2),
        quotient_over_mk(normal_subgroup_quotient_relation(s), b)
    )
} by {
    if is_normal_subgroup(s) and normal_subgroup_rel(s, a1, a2) {
        normal_subgroup_quotient_relation_rel(s)
        normal_subgroup_quotient_relation(s).rel(a1, a2)
        normal_subgroup_quotient_relation_respects_mul(s)
        quotient_over_binary_op_projection_left(normal_subgroup_quotient_relation(s), G.mul, a1, a2, b)
    }
}

/// Multiplication in the normal-subgroup quotient respects related right representatives.
theorem normal_subgroup_quotient_mul_projection_right[G: Group](s: Subgroup[G], a: G, b1: G, b2: G) {
    is_normal_subgroup(s) and normal_subgroup_rel(s, b1, b2) implies
    normal_subgroup_quotient_mul(s,
        quotient_over_mk(normal_subgroup_quotient_relation(s), a),
        quotient_over_mk(normal_subgroup_quotient_relation(s), b1)
    ) = normal_subgroup_quotient_mul(s,
        quotient_over_mk(normal_subgroup_quotient_relation(s), a),
        quotient_over_mk(normal_subgroup_quotient_relation(s), b2)
    )
} by {
    if is_normal_subgroup(s) and normal_subgroup_rel(s, b1, b2) {
        normal_subgroup_quotient_relation_rel(s)
        normal_subgroup_quotient_relation(s).rel(b1, b2)
        normal_subgroup_quotient_relation_respects_mul(s)
        quotient_over_binary_op_projection_right(normal_subgroup_quotient_relation(s), G.mul, a, b1, b2)
    }
}

/// Inversion on the relation-indexed quotient by a normal subgroup.
define normal_subgroup_quotient_inverse[G: Group](s: Subgroup[G]) ->
    QuotientOver[G] -> QuotientOver[G] {
    quotient_over_unary_op(normal_subgroup_quotient_relation(s), G.inverse)
}

/// Inversion in the normal-subgroup quotient agrees with inversion of representatives.
theorem normal_subgroup_quotient_inverse_projection[G: Group](s: Subgroup[G], a: G) {
    is_normal_subgroup(s) implies
    normal_subgroup_quotient_inverse(s, quotient_over_mk(normal_subgroup_quotient_relation(s), a)) =
        quotient_over_mk(normal_subgroup_quotient_relation(s), a.inverse)
} by {
    if is_normal_subgroup(s) {
        normal_subgroup_quotient_relation_respects_inverse(s)
        quotient_over_unary_op_projection(normal_subgroup_quotient_relation(s), G.inverse, a)
        G.inverse(a) = a.inverse
    }
}

/// Inversion in the normal-subgroup quotient respects related representatives.
theorem normal_subgroup_quotient_inverse_projection_compatible[G: Group](s: Subgroup[G], a: G, b: G) {
    is_normal_subgroup(s) and normal_subgroup_rel(s, a, b) implies
    normal_subgroup_quotient_inverse(s, quotient_over_mk(normal_subgroup_quotient_relation(s), a)) =
        normal_subgroup_quotient_inverse(s, quotient_over_mk(normal_subgroup_quotient_relation(s), b))
} by {
    if is_normal_subgroup(s) and normal_subgroup_rel(s, a, b) {
        normal_subgroup_quotient_relation_rel(s)
        normal_subgroup_quotient_relation(s).rel(a, b)
        normal_subgroup_quotient_relation_respects_inverse(s)
        quotient_over_unary_op_projection_compatible(normal_subgroup_quotient_relation(s), G.inverse, a, b)
    }
}

/// Inversion twice on a projected normal-subgroup quotient representative is the original representative.
theorem normal_subgroup_quotient_inverse_inverse[G: Group](s: Subgroup[G], a: G) {
    is_normal_subgroup(s) implies
    normal_subgroup_quotient_inverse(s,
        normal_subgroup_quotient_inverse(s, quotient_over_mk(normal_subgroup_quotient_relation(s), a))) =
        quotient_over_mk(normal_subgroup_quotient_relation(s), a)
} by {
    if is_normal_subgroup(s) {
        normal_subgroup_quotient_inverse_projection(s, a)
        normal_subgroup_quotient_inverse_projection(s, a.inverse)
        inverse_inverse(a)
        normal_subgroup_quotient_inverse(s,
            normal_subgroup_quotient_inverse(s, quotient_over_mk(normal_subgroup_quotient_relation(s), a))) =
            quotient_over_mk(normal_subgroup_quotient_relation(s), a)
    }
}

/// The identity element of the normal-subgroup quotient.
define normal_subgroup_quotient_one[G: Group](s: Subgroup[G]) -> QuotientOver[G] {
    quotient_over_mk(normal_subgroup_quotient_relation(s), G.1)
}

/// The identity element of the normal-subgroup quotient is the projection of the group identity.
theorem normal_subgroup_quotient_one_projection[G: Group](s: Subgroup[G]) {
    normal_subgroup_quotient_one(s) = quotient_over_mk(normal_subgroup_quotient_relation(s), G.1)
}

/// The canonical projection from a group to its quotient by a normal subgroup.
define normal_subgroup_quotient_project[G: Group](s: Subgroup[G], a: G) -> QuotientOver[G] {
    quotient_over_mk(normal_subgroup_quotient_relation(s), a)
}

/// The canonical projection is the underlying quotient projection.
theorem normal_subgroup_quotient_project_eq_mk[G: Group](s: Subgroup[G], a: G) {
    normal_subgroup_quotient_project(s, a) = quotient_over_mk(normal_subgroup_quotient_relation(s), a)
}

/// Related representatives have the same canonical projection.
theorem normal_subgroup_quotient_project_eq_of_rel[G: Group](s: Subgroup[G], a: G, b: G) {
    normal_subgroup_rel(s, a, b) implies
    normal_subgroup_quotient_project(s, a) = normal_subgroup_quotient_project(s, b)
} by {
    if normal_subgroup_rel(s, a, b) {
        normal_subgroup_quotient_relation_rel(s)
        normal_subgroup_quotient_relation(s).rel(a, b)
        quotient_over_mk_eq_of_rel(normal_subgroup_quotient_relation(s), a, b)
        normal_subgroup_quotient_project(s, a) = normal_subgroup_quotient_project(s, b)
    }
}

/// Equality of canonical projections gives the normal-subgroup relation.
theorem normal_subgroup_rel_of_quotient_project_eq[G: Group](s: Subgroup[G], a: G, b: G) {
    normal_subgroup_quotient_project(s, a) = normal_subgroup_quotient_project(s, b)
    implies normal_subgroup_rel(s, a, b)
} by {
    if normal_subgroup_quotient_project(s, a) = normal_subgroup_quotient_project(s, b) {
        rel_of_quotient_over_mk_eq(normal_subgroup_quotient_relation(s), a, b)
        normal_subgroup_quotient_relation_rel(s)
        normal_subgroup_quotient_relation(s).rel(a, b) = normal_subgroup_rel(s, a, b)
        normal_subgroup_rel(s, a, b)
    }
}

/// Equality of canonical projections is exactly the normal-subgroup relation.
theorem normal_subgroup_quotient_project_eq_iff_rel[G: Group](s: Subgroup[G], a: G, b: G) {
    (normal_subgroup_quotient_project(s, a) = normal_subgroup_quotient_project(s, b)) =
    normal_subgroup_rel(s, a, b)
} by {
    if normal_subgroup_quotient_project(s, a) = normal_subgroup_quotient_project(s, b) {
        normal_subgroup_rel_of_quotient_project_eq(s, a, b)
        normal_subgroup_rel(s, a, b)
    }
    if normal_subgroup_rel(s, a, b) {
        normal_subgroup_quotient_project_eq_of_rel(s, a, b)
        normal_subgroup_quotient_project(s, a) = normal_subgroup_quotient_project(s, b)
    }
    quotient_over_mk_eq_iff_rel(normal_subgroup_quotient_relation(s), a, b)
    normal_subgroup_quotient_relation_rel(s)
}

/// The canonical projection sends the group identity to the quotient identity.
theorem normal_subgroup_quotient_project_one[G: Group](s: Subgroup[G]) {
    normal_subgroup_quotient_project(s, G.1) = normal_subgroup_quotient_one(s)
} by {
    normal_subgroup_quotient_one_projection(s)
}

/// Relation to the identity is membership in the subgroup.
theorem normal_subgroup_rel_one_right[G: Group](s: Subgroup[G], a: G) {
    normal_subgroup_rel(s, a, G.1) = s.contains(a)
} by {
    s.contains(a * G.1.inverse) = s.contains(a)
}

/// Relation from the identity is membership in the subgroup.
theorem normal_subgroup_rel_one_left[G: Group](s: Subgroup[G], a: G) {
    normal_subgroup_rel(s, G.1, a) = s.contains(a)
} by {
    subgroup_contains_inverse_iff(s, a)
}

/// Membership of the left quotient difference gives the normal-subgroup relation.
theorem normal_subgroup_rel_of_contains_inverse_mul[G: Group](s: Subgroup[G], a: G, b: G) {
    is_normal_subgroup(s) and s.contains(b.inverse * a) implies normal_subgroup_rel(s, a, b)
} by {
    if is_normal_subgroup(s) and s.contains(b.inverse * a) {
        normal_conjugate_mem(s, b, b.inverse * a)
        (b * b.inverse) * a * b.inverse = G.1 * a * b.inverse
        b * (b.inverse * a) * b.inverse = a * b.inverse
        normal_subgroup_rel(s, a, b)
    }
}

/// The normal-subgroup relation gives membership of the left quotient difference.
theorem normal_subgroup_contains_inverse_mul_of_rel[G: Group](s: Subgroup[G], a: G, b: G) {
    is_normal_subgroup(s) and normal_subgroup_rel(s, a, b) implies s.contains(b.inverse * a)
} by {
    if is_normal_subgroup(s) and normal_subgroup_rel(s, a, b) {
        normal_conjugate_mem(s, b.inverse, a * b.inverse)
        s.contains(b.inverse * (a * b.inverse) * b.inverse.inverse)
        inverse_inverse(b)
        b.inverse * a * (b.inverse * b) = b.inverse * a * G.1
        s.contains(b.inverse * a)
    }
}

/// Membership of the left quotient difference is the normal-subgroup relation.
theorem normal_subgroup_contains_inverse_mul_iff_rel[G: Group](s: Subgroup[G], a: G, b: G) {
    is_normal_subgroup(s) implies
    s.contains(b.inverse * a) = normal_subgroup_rel(s, a, b)
} by {
    if is_normal_subgroup(s) {
        if s.contains(b.inverse * a) {
            normal_subgroup_rel_of_contains_inverse_mul(s, a, b)
        }
        if normal_subgroup_rel(s, a, b) {
            normal_subgroup_contains_inverse_mul_of_rel(s, a, b)
        }
    }
}

/// A representative projects to quotient identity exactly when it lies in the subgroup.
theorem normal_subgroup_quotient_project_eq_one_iff_contains[G: Group](s: Subgroup[G], a: G) {
    (normal_subgroup_quotient_project(s, a) = normal_subgroup_quotient_one(s)) = s.contains(a)
} by {
    normal_subgroup_quotient_project_one(s)
    normal_subgroup_quotient_project_eq_iff_rel(s, a, G.1)
    normal_subgroup_rel_one_right(s, a)
}

/// Quotient identity equals a representative projection exactly when the representative lies in the subgroup.
theorem normal_subgroup_quotient_one_eq_project_iff_contains[G: Group](s: Subgroup[G], a: G) {
    (normal_subgroup_quotient_one(s) = normal_subgroup_quotient_project(s, a)) = s.contains(a)
} by {
    normal_subgroup_quotient_project_one(s)
    normal_subgroup_quotient_project_eq_iff_rel(s, G.1, a)
    normal_subgroup_rel_one_left(s, a)
}

/// Subgroup membership gives equality with quotient identity.
theorem normal_subgroup_quotient_project_eq_one_of_contains[G: Group](s: Subgroup[G], a: G) {
    s.contains(a) implies normal_subgroup_quotient_project(s, a) = normal_subgroup_quotient_one(s)
} by {
    if s.contains(a) {
        normal_subgroup_quotient_project_eq_one_iff_contains(s, a)
    }
}

/// Equality with quotient identity gives subgroup membership.
theorem normal_subgroup_contains_of_quotient_project_eq_one[G: Group](s: Subgroup[G], a: G) {
    normal_subgroup_quotient_project(s, a) = normal_subgroup_quotient_one(s) implies s.contains(a)
} by {
    if normal_subgroup_quotient_project(s, a) = normal_subgroup_quotient_one(s) {
        normal_subgroup_quotient_project_eq_one_iff_contains(s, a)
    }
}

/// Subgroup membership gives equality from quotient identity.
theorem normal_subgroup_quotient_one_eq_project_of_contains[G: Group](s: Subgroup[G], a: G) {
    s.contains(a) implies normal_subgroup_quotient_one(s) = normal_subgroup_quotient_project(s, a)
} by {
    if s.contains(a) {
        normal_subgroup_quotient_one_eq_project_iff_contains(s, a)
    }
}

/// Equality from quotient identity gives subgroup membership.
theorem normal_subgroup_contains_of_quotient_one_eq_project[G: Group](s: Subgroup[G], a: G) {
    normal_subgroup_quotient_one(s) = normal_subgroup_quotient_project(s, a) implies s.contains(a)
} by {
    if normal_subgroup_quotient_one(s) = normal_subgroup_quotient_project(s, a) {
        normal_subgroup_quotient_one_eq_project_iff_contains(s, a)
    }
}

/// The canonical projection preserves multiplication.
theorem normal_subgroup_quotient_project_mul[G: Group](s: Subgroup[G], a: G, b: G) {
    is_normal_subgroup(s) implies
    normal_subgroup_quotient_project(s, a * b) =
    normal_subgroup_quotient_mul(s,
        normal_subgroup_quotient_project(s, a),
        normal_subgroup_quotient_project(s, b))
} by {
    if is_normal_subgroup(s) {
        normal_subgroup_quotient_mul_projection(s, a, b)
        normal_subgroup_quotient_project(s, a * b) =
        normal_subgroup_quotient_mul(s,
            normal_subgroup_quotient_project(s, a),
            normal_subgroup_quotient_project(s, b))
    }
}

/// The canonical projection preserves inversion.
theorem normal_subgroup_quotient_project_inverse[G: Group](s: Subgroup[G], a: G) {
    is_normal_subgroup(s) implies
    normal_subgroup_quotient_project(s, a.inverse) =
    normal_subgroup_quotient_inverse(s, normal_subgroup_quotient_project(s, a))
} by {
    if is_normal_subgroup(s) {
        normal_subgroup_quotient_inverse_projection(s, a)
        normal_subgroup_quotient_project(s, a.inverse) =
            normal_subgroup_quotient_inverse(s, normal_subgroup_quotient_project(s, a))
    }
}

/// Quotient multiplication of canonical projections respects related left representatives.
theorem normal_subgroup_quotient_project_mul_left_of_rel[G: Group](
    s: Subgroup[G], a1: G, a2: G, b: G
) {
    is_normal_subgroup(s) and normal_subgroup_rel(s, a1, a2) implies
    normal_subgroup_quotient_mul(s,
        normal_subgroup_quotient_project(s, a1),
        normal_subgroup_quotient_project(s, b)) =
    normal_subgroup_quotient_mul(s,
        normal_subgroup_quotient_project(s, a2),
        normal_subgroup_quotient_project(s, b))
} by {
    if is_normal_subgroup(s) and normal_subgroup_rel(s, a1, a2) {
        normal_subgroup_quotient_mul_projection_left(s, a1, a2, b)
    }
}

/// Quotient multiplication of canonical projections respects related right representatives.
theorem normal_subgroup_quotient_project_mul_right_of_rel[G: Group](
    s: Subgroup[G], a: G, b1: G, b2: G
) {
    is_normal_subgroup(s) and normal_subgroup_rel(s, b1, b2) implies
    normal_subgroup_quotient_mul(s,
        normal_subgroup_quotient_project(s, a),
        normal_subgroup_quotient_project(s, b1)) =
    normal_subgroup_quotient_mul(s,
        normal_subgroup_quotient_project(s, a),
        normal_subgroup_quotient_project(s, b2))
} by {
    if is_normal_subgroup(s) and normal_subgroup_rel(s, b1, b2) {
        normal_subgroup_quotient_mul_projection_right(s, a, b1, b2)
    }
}

/// Quotient inversion of canonical projections respects related representatives.
theorem normal_subgroup_quotient_project_inverse_of_rel[G: Group](s: Subgroup[G], a: G, b: G) {
    is_normal_subgroup(s) and normal_subgroup_rel(s, a, b) implies
    normal_subgroup_quotient_inverse(s, normal_subgroup_quotient_project(s, a)) =
    normal_subgroup_quotient_inverse(s, normal_subgroup_quotient_project(s, b))
} by {
    if is_normal_subgroup(s) and normal_subgroup_rel(s, a, b) {
        normal_subgroup_quotient_inverse_projection_compatible(s, a, b)
    }
}

/// The projected quotient difference is identity exactly when the representatives are related.
theorem normal_subgroup_quotient_project_mul_inverse_eq_one_iff_rel[G: Group](
    s: Subgroup[G], a: G, b: G
) {
    is_normal_subgroup(s) implies
    (normal_subgroup_quotient_mul(s,
        normal_subgroup_quotient_project(s, a),
        normal_subgroup_quotient_inverse(s, normal_subgroup_quotient_project(s, b))) =
        normal_subgroup_quotient_one(s)) = normal_subgroup_rel(s, a, b)
} by {
    if is_normal_subgroup(s) {
        normal_subgroup_quotient_project_inverse(s, b)
        normal_subgroup_quotient_project_mul(s, a, b.inverse)
        normal_subgroup_quotient_project_eq_one_iff_contains(s, a * b.inverse)
        (normal_subgroup_quotient_mul(s,
            normal_subgroup_quotient_project(s, a),
            normal_subgroup_quotient_inverse(s, normal_subgroup_quotient_project(s, b))) =
            normal_subgroup_quotient_one(s)) = normal_subgroup_rel(s, a, b)
    }
}

/// Related representatives have projected quotient difference equal to identity.
theorem normal_subgroup_quotient_project_mul_inverse_eq_one_of_rel[G: Group](
    s: Subgroup[G], a: G, b: G
) {
    is_normal_subgroup(s) and normal_subgroup_rel(s, a, b) implies
    normal_subgroup_quotient_mul(s,
        normal_subgroup_quotient_project(s, a),
        normal_subgroup_quotient_inverse(s, normal_subgroup_quotient_project(s, b))) =
        normal_subgroup_quotient_one(s)
} by {
    if is_normal_subgroup(s) and normal_subgroup_rel(s, a, b) {
        normal_subgroup_quotient_project_mul_inverse_eq_one_iff_rel(s, a, b)
    }
}

/// Projected quotient difference equal to identity gives related representatives.
theorem normal_subgroup_rel_of_quotient_project_mul_inverse_eq_one[G: Group](
    s: Subgroup[G], a: G, b: G
) {
    is_normal_subgroup(s) and
    normal_subgroup_quotient_mul(s,
        normal_subgroup_quotient_project(s, a),
        normal_subgroup_quotient_inverse(s, normal_subgroup_quotient_project(s, b))) =
        normal_subgroup_quotient_one(s) implies
    normal_subgroup_rel(s, a, b)
} by {
    if is_normal_subgroup(s) and
        normal_subgroup_quotient_mul(s,
            normal_subgroup_quotient_project(s, a),
            normal_subgroup_quotient_inverse(s, normal_subgroup_quotient_project(s, b))) =
            normal_subgroup_quotient_one(s) {
        normal_subgroup_quotient_project_mul_inverse_eq_one_iff_rel(s, a, b)
    }
}

/// The projected quotient difference is identity exactly when the representative difference lies in the subgroup.
theorem normal_subgroup_quotient_project_mul_inverse_eq_one_iff_contains[G: Group](
    s: Subgroup[G], a: G, b: G
) {
    is_normal_subgroup(s) implies
    (normal_subgroup_quotient_mul(s,
        normal_subgroup_quotient_project(s, a),
        normal_subgroup_quotient_inverse(s, normal_subgroup_quotient_project(s, b))) =
        normal_subgroup_quotient_one(s)) = s.contains(a * b.inverse)
} by {
    if is_normal_subgroup(s) {
        normal_subgroup_quotient_project_mul_inverse_eq_one_iff_rel(s, a, b)
        (normal_subgroup_quotient_mul(s,
            normal_subgroup_quotient_project(s, a),
            normal_subgroup_quotient_inverse(s, normal_subgroup_quotient_project(s, b))) =
            normal_subgroup_quotient_one(s)) = s.contains(a * b.inverse)
    }
}

/// Subgroup membership of the representative difference gives projected quotient difference identity.
theorem normal_subgroup_quotient_project_mul_inverse_eq_one_of_contains[G: Group](
    s: Subgroup[G], a: G, b: G
) {
    is_normal_subgroup(s) and s.contains(a * b.inverse) implies
    normal_subgroup_quotient_mul(s,
        normal_subgroup_quotient_project(s, a),
        normal_subgroup_quotient_inverse(s, normal_subgroup_quotient_project(s, b))) =
        normal_subgroup_quotient_one(s)
} by {
    if is_normal_subgroup(s) and s.contains(a * b.inverse) {
        normal_subgroup_quotient_project_mul_inverse_eq_one_iff_contains(s, a, b)
    }
}

/// Projected quotient difference identity gives subgroup membership of the representative difference.
theorem normal_subgroup_contains_mul_inverse_of_quotient_project_mul_inverse_eq_one[G: Group](
    s: Subgroup[G], a: G, b: G
) {
    is_normal_subgroup(s) and
    normal_subgroup_quotient_mul(s,
        normal_subgroup_quotient_project(s, a),
        normal_subgroup_quotient_inverse(s, normal_subgroup_quotient_project(s, b))) =
        normal_subgroup_quotient_one(s) implies
    s.contains(a * b.inverse)
} by {
    if is_normal_subgroup(s) and
        normal_subgroup_quotient_mul(s,
            normal_subgroup_quotient_project(s, a),
            normal_subgroup_quotient_inverse(s, normal_subgroup_quotient_project(s, b))) =
            normal_subgroup_quotient_one(s) {
        normal_subgroup_quotient_project_mul_inverse_eq_one_iff_contains(s, a, b)
    }
}

/// A projected representative multiplied by a projected inverse is the projection of the quotient difference.
theorem normal_subgroup_quotient_project_mul_inverse_projection[G: Group](
    s: Subgroup[G], a: G, b: G
) {
    is_normal_subgroup(s) implies
    normal_subgroup_quotient_mul(s,
        normal_subgroup_quotient_project(s, a),
        normal_subgroup_quotient_inverse(s, normal_subgroup_quotient_project(s, b))) =
        normal_subgroup_quotient_project(s, a * b.inverse)
} by {
    if is_normal_subgroup(s) {
        normal_subgroup_quotient_project_inverse(s, b)
        normal_subgroup_quotient_inverse(s, normal_subgroup_quotient_project(s, b)) =
            normal_subgroup_quotient_project(s, b.inverse)
        normal_subgroup_quotient_project_mul(s, a, b.inverse)
        normal_subgroup_quotient_mul(s,
            normal_subgroup_quotient_project(s, a),
            normal_subgroup_quotient_inverse(s, normal_subgroup_quotient_project(s, b))) =
            normal_subgroup_quotient_project(s, a * b.inverse)
    }
}

/// A projected inverse multiplied by a projected representative is the projection of the left quotient difference.
theorem normal_subgroup_quotient_project_inverse_mul_projection[G: Group](
    s: Subgroup[G], a: G, b: G
) {
    is_normal_subgroup(s) implies
    normal_subgroup_quotient_mul(s,
        normal_subgroup_quotient_inverse(s, normal_subgroup_quotient_project(s, b)),
        normal_subgroup_quotient_project(s, a)) =
        normal_subgroup_quotient_project(s, b.inverse * a)
} by {
    if is_normal_subgroup(s) {
        normal_subgroup_quotient_project_inverse(s, b)
        normal_subgroup_quotient_inverse(s, normal_subgroup_quotient_project(s, b)) =
            normal_subgroup_quotient_project(s, b.inverse)
        normal_subgroup_quotient_project_mul(s, b.inverse, a)
        normal_subgroup_quotient_mul(s,
            normal_subgroup_quotient_inverse(s, normal_subgroup_quotient_project(s, b)),
            normal_subgroup_quotient_project(s, a)) =
            normal_subgroup_quotient_project(s, b.inverse * a)
    }
}

/// The left projected quotient difference is identity exactly when the representatives are related.
theorem normal_subgroup_quotient_project_inverse_mul_eq_one_iff_rel[G: Group](
    s: Subgroup[G], a: G, b: G
) {
    is_normal_subgroup(s) implies
    (normal_subgroup_quotient_mul(s,
        normal_subgroup_quotient_inverse(s, normal_subgroup_quotient_project(s, b)),
        normal_subgroup_quotient_project(s, a)) =
        normal_subgroup_quotient_one(s)) = normal_subgroup_rel(s, a, b)
} by {
    if is_normal_subgroup(s) {
        normal_subgroup_quotient_project_inverse_mul_projection(s, a, b)
        normal_subgroup_quotient_project_eq_one_iff_contains(s, b.inverse * a)
        normal_subgroup_contains_inverse_mul_iff_rel(s, a, b)
        (normal_subgroup_quotient_mul(s,
            normal_subgroup_quotient_inverse(s, normal_subgroup_quotient_project(s, b)),
            normal_subgroup_quotient_project(s, a)) =
            normal_subgroup_quotient_one(s)) = normal_subgroup_rel(s, a, b)
    }
}

/// Related representatives have left projected quotient difference equal to identity.
theorem normal_subgroup_quotient_project_inverse_mul_eq_one_of_rel[G: Group](
    s: Subgroup[G], a: G, b: G
) {
    is_normal_subgroup(s) and normal_subgroup_rel(s, a, b) implies
    normal_subgroup_quotient_mul(s,
        normal_subgroup_quotient_inverse(s, normal_subgroup_quotient_project(s, b)),
        normal_subgroup_quotient_project(s, a)) =
        normal_subgroup_quotient_one(s)
} by {
    if is_normal_subgroup(s) and normal_subgroup_rel(s, a, b) {
        normal_subgroup_quotient_project_inverse_mul_eq_one_iff_rel(s, a, b)
    }
}

/// Left projected quotient difference equal to identity gives related representatives.
theorem normal_subgroup_rel_of_quotient_project_inverse_mul_eq_one[G: Group](
    s: Subgroup[G], a: G, b: G
) {
    is_normal_subgroup(s) and
    normal_subgroup_quotient_mul(s,
        normal_subgroup_quotient_inverse(s, normal_subgroup_quotient_project(s, b)),
        normal_subgroup_quotient_project(s, a)) =
        normal_subgroup_quotient_one(s) implies
    normal_subgroup_rel(s, a, b)
} by {
    if is_normal_subgroup(s) and
        normal_subgroup_quotient_mul(s,
            normal_subgroup_quotient_inverse(s, normal_subgroup_quotient_project(s, b)),
            normal_subgroup_quotient_project(s, a)) =
            normal_subgroup_quotient_one(s) {
        normal_subgroup_quotient_project_inverse_mul_eq_one_iff_rel(s, a, b)
    }
}

/// The left projected quotient difference is identity exactly when the left representative difference lies in the subgroup.
theorem normal_subgroup_quotient_project_inverse_mul_eq_one_iff_contains[G: Group](
    s: Subgroup[G], a: G, b: G
) {
    is_normal_subgroup(s) implies
    (normal_subgroup_quotient_mul(s,
        normal_subgroup_quotient_inverse(s, normal_subgroup_quotient_project(s, b)),
        normal_subgroup_quotient_project(s, a)) =
        normal_subgroup_quotient_one(s)) = s.contains(b.inverse * a)
} by {
    if is_normal_subgroup(s) {
        normal_subgroup_quotient_project_inverse_mul_eq_one_iff_rel(s, a, b)
        normal_subgroup_contains_inverse_mul_iff_rel(s, a, b)
        s.contains(b.inverse * a) = normal_subgroup_rel(s, a, b)
        (normal_subgroup_quotient_mul(s,
            normal_subgroup_quotient_inverse(s, normal_subgroup_quotient_project(s, b)),
            normal_subgroup_quotient_project(s, a)) =
            normal_subgroup_quotient_one(s)) = s.contains(b.inverse * a)
    }
}

/// Subgroup membership of the left representative difference gives left projected quotient difference identity.
theorem normal_subgroup_quotient_project_inverse_mul_eq_one_of_contains[G: Group](
    s: Subgroup[G], a: G, b: G
) {
    is_normal_subgroup(s) and s.contains(b.inverse * a) implies
    normal_subgroup_quotient_mul(s,
        normal_subgroup_quotient_inverse(s, normal_subgroup_quotient_project(s, b)),
        normal_subgroup_quotient_project(s, a)) =
        normal_subgroup_quotient_one(s)
} by {
    if is_normal_subgroup(s) and s.contains(b.inverse * a) {
        normal_subgroup_quotient_project_inverse_mul_eq_one_iff_contains(s, a, b)
    }
}

/// Left projected quotient difference identity gives subgroup membership of the left representative difference.
theorem normal_subgroup_contains_inverse_mul_of_quotient_project_inverse_mul_eq_one[G: Group](
    s: Subgroup[G], a: G, b: G
) {
    is_normal_subgroup(s) and
    normal_subgroup_quotient_mul(s,
        normal_subgroup_quotient_inverse(s, normal_subgroup_quotient_project(s, b)),
        normal_subgroup_quotient_project(s, a)) =
        normal_subgroup_quotient_one(s) implies
    s.contains(b.inverse * a)
} by {
    if is_normal_subgroup(s) and
        normal_subgroup_quotient_mul(s,
            normal_subgroup_quotient_inverse(s, normal_subgroup_quotient_project(s, b)),
            normal_subgroup_quotient_project(s, a)) =
            normal_subgroup_quotient_one(s) {
        normal_subgroup_quotient_project_inverse_mul_eq_one_iff_contains(s, a, b)
    }
}

/// Powers on the quotient by a normal subgroup.
define normal_subgroup_quotient_pow[G: Group](
    s: Subgroup[G], q: QuotientOver[G], n: Nat
) -> QuotientOver[G] {
    match n {
        Nat.zero {
            normal_subgroup_quotient_one(s)
        }
        Nat.suc(k) {
            normal_subgroup_quotient_mul(s, q, normal_subgroup_quotient_pow(s, q, k))
        }
    }
}

/// The zeroth power on a normal-subgroup quotient is the quotient identity.
theorem normal_subgroup_quotient_pow_zero_projection[G: Group](s: Subgroup[G], q: QuotientOver[G]) {
    normal_subgroup_quotient_pow(s, q, Nat.0) = normal_subgroup_quotient_one(s)
} by {
}

/// The successor power on a normal-subgroup quotient multiplies by the base.
theorem normal_subgroup_quotient_pow_suc_projection[G: Group](
    s: Subgroup[G], q: QuotientOver[G], n: Nat
) {
    normal_subgroup_quotient_pow(s, q, n.suc) =
        normal_subgroup_quotient_mul(s, q, normal_subgroup_quotient_pow(s, q, n))
}

/// The zeroth power on a projected normal-subgroup representative is the projected identity.
theorem normal_subgroup_quotient_pow_mk_zero[G: Group](s: Subgroup[G], a: G) {
    normal_subgroup_quotient_pow(s, quotient_over_mk(normal_subgroup_quotient_relation(s), a), Nat.0) =
        quotient_over_mk(normal_subgroup_quotient_relation(s), a.pow(Nat.0))
} by {
    normal_subgroup_quotient_pow_zero_projection(s, quotient_over_mk(normal_subgroup_quotient_relation(s), a))
    normal_subgroup_quotient_one_projection(s)
}

/// Projected powers in a normal-subgroup quotient advance through successors.
theorem normal_subgroup_quotient_pow_mk_suc_step[G: Group](s: Subgroup[G], a: G, n: Nat) {
    is_normal_subgroup(s) and
    normal_subgroup_quotient_pow(s, quotient_over_mk(normal_subgroup_quotient_relation(s), a), n) =
        quotient_over_mk(normal_subgroup_quotient_relation(s), a.pow(n))
    implies
    normal_subgroup_quotient_pow(s, quotient_over_mk(normal_subgroup_quotient_relation(s), a), n.suc) =
        quotient_over_mk(normal_subgroup_quotient_relation(s), a.pow(n.suc))
} by {
    if is_normal_subgroup(s) and
    normal_subgroup_quotient_pow(s, quotient_over_mk(normal_subgroup_quotient_relation(s), a), n) =
        quotient_over_mk(normal_subgroup_quotient_relation(s), a.pow(n)) {
        normal_subgroup_quotient_pow_suc_projection(
            s, quotient_over_mk(normal_subgroup_quotient_relation(s), a), n)
        normal_subgroup_quotient_mul_projection(s, a, a.pow(n))
        normal_subgroup_quotient_pow(s, quotient_over_mk(normal_subgroup_quotient_relation(s), a), n.suc) =
            quotient_over_mk(normal_subgroup_quotient_relation(s), a.pow(n.suc))
    }
}

/// The first power of a projected representative is the projected first power.
theorem normal_subgroup_quotient_pow_mk_one[G: Group](s: Subgroup[G], a: G) {
    is_normal_subgroup(s) implies
    normal_subgroup_quotient_pow(s, quotient_over_mk(normal_subgroup_quotient_relation(s), a), Nat.1) =
        quotient_over_mk(normal_subgroup_quotient_relation(s), a.pow(Nat.1))
} by {
    if is_normal_subgroup(s) {
        normal_subgroup_quotient_pow_mk_zero(s, a)
        normal_subgroup_quotient_pow_mk_suc_step(s, a, Nat.0)
    }
}

/// The zeroth power of a projected representative is the projected group zeroth power.
theorem normal_subgroup_quotient_project_pow_zero[G: Group](s: Subgroup[G], a: G) {
    normal_subgroup_quotient_pow(s, normal_subgroup_quotient_project(s, a), Nat.0) =
        normal_subgroup_quotient_project(s, a.pow(Nat.0))
} by {
    normal_subgroup_quotient_pow_mk_zero(s, a)
}

/// The first power of a projected representative is the projected group first power.
theorem normal_subgroup_quotient_project_pow_one[G: Group](s: Subgroup[G], a: G) {
    is_normal_subgroup(s) implies
    normal_subgroup_quotient_pow(s, normal_subgroup_quotient_project(s, a), Nat.1) =
        normal_subgroup_quotient_project(s, a.pow(Nat.1))
} by {
    if is_normal_subgroup(s) {
        normal_subgroup_quotient_pow_mk_one(s, a)
        normal_subgroup_quotient_pow(s, normal_subgroup_quotient_project(s, a), Nat.1) =
            normal_subgroup_quotient_project(s, a.pow(Nat.1))
    }
}

/// Projected powers in a normal-subgroup quotient advance through successors.
theorem normal_subgroup_quotient_project_pow_suc_step[G: Group](s: Subgroup[G], a: G, n: Nat) {
    is_normal_subgroup(s) and
    normal_subgroup_quotient_pow(s, normal_subgroup_quotient_project(s, a), n) =
        normal_subgroup_quotient_project(s, a.pow(n))
    implies
    normal_subgroup_quotient_pow(s, normal_subgroup_quotient_project(s, a), n.suc) =
        normal_subgroup_quotient_project(s, a.pow(n.suc))
} by {
    if is_normal_subgroup(s) and
    normal_subgroup_quotient_pow(s, normal_subgroup_quotient_project(s, a), n) =
        normal_subgroup_quotient_project(s, a.pow(n)) {
        normal_subgroup_quotient_pow_mk_suc_step(s, a, n)
        normal_subgroup_quotient_pow(s,
            quotient_over_mk(normal_subgroup_quotient_relation(s), a), n.suc) =
            quotient_over_mk(normal_subgroup_quotient_relation(s), a.pow(n.suc))
        normal_subgroup_quotient_pow(s, normal_subgroup_quotient_project(s, a), n.suc) =
            normal_subgroup_quotient_project(s, a.pow(n.suc))
    }
}

/// The zeroth power of a projected representative is the quotient identity.
theorem normal_subgroup_quotient_project_pow_zero_eq_one[G: Group](s: Subgroup[G], a: G) {
    normal_subgroup_quotient_pow(s, normal_subgroup_quotient_project(s, a), Nat.0) =
        normal_subgroup_quotient_one(s)
} by {
    normal_subgroup_quotient_pow_zero_projection(s, normal_subgroup_quotient_project(s, a))
}

/// True if a normal-subgroup quotient power of a representative is the projected group power.
define normal_subgroup_quotient_projected_power_law[G: Group](
    s: Subgroup[G], a: G, n: Nat
) -> Bool {
    normal_subgroup_quotient_pow(s, quotient_over_mk(normal_subgroup_quotient_relation(s), a), n) =
        quotient_over_mk(normal_subgroup_quotient_relation(s), a.pow(n))
}

/// Projected powers on a normal-subgroup quotient agree with the projected group power for every exponent.
theorem normal_subgroup_quotient_pow_mk_all[G: Group](s: Subgroup[G], a: G, n: Nat) {
    is_normal_subgroup(s) implies
    normal_subgroup_quotient_projected_power_law(s, a, n)
} by {
    define g(k: Nat) -> Bool {
        is_normal_subgroup(s) implies normal_subgroup_quotient_projected_power_law(s, a, k)
    }
    normal_subgroup_quotient_pow_mk_zero(s, a)
    g(Nat.0)
    forall(k: Nat) {
        if g(k) {
            if is_normal_subgroup(s) {
                normal_subgroup_quotient_pow_mk_suc_step(s, a, k)
                normal_subgroup_quotient_pow(s,
                    quotient_over_mk(normal_subgroup_quotient_relation(s), a), k.suc) =
                    quotient_over_mk(normal_subgroup_quotient_relation(s), a.pow(k.suc))
                normal_subgroup_quotient_projected_power_law(s, a, k.suc)
            }
            g(k.suc)
        }
    }
    g(Nat.0) and forall(k: Nat) { g(k) implies g(k.suc) }
    alt_induction(g)
    forall(k: Nat) { g(k) }
    if is_normal_subgroup(s) {
        normal_subgroup_quotient_projected_power_law(s, a, n)
    }
}

/// The projection-facing alias: projected powers in a normal-subgroup quotient agree with the projected group power.
theorem normal_subgroup_quotient_project_pow_all[G: Group](s: Subgroup[G], a: G, n: Nat) {
    is_normal_subgroup(s) implies
    normal_subgroup_quotient_pow(s, normal_subgroup_quotient_project(s, a), n) =
        normal_subgroup_quotient_project(s, a.pow(n))
} by {
    if is_normal_subgroup(s) {
        normal_subgroup_quotient_pow_mk_all(s, a, n)
        normal_subgroup_quotient_pow(s,
            quotient_over_mk(normal_subgroup_quotient_relation(s), a), n) =
            quotient_over_mk(normal_subgroup_quotient_relation(s), a.pow(n))
        normal_subgroup_quotient_pow(s, normal_subgroup_quotient_project(s, a), n) =
            normal_subgroup_quotient_project(s, a.pow(n))
    }
}

/// Projected powers add exponents: the projection of `a^(n+m)` equals the product of the projections of `a^n` and `a^m`.
theorem normal_subgroup_quotient_project_pow_add[G: Group](s: Subgroup[G], a: G, n: Nat, m: Nat) {
    is_normal_subgroup(s) implies
    normal_subgroup_quotient_pow(s, normal_subgroup_quotient_project(s, a), n + m) =
        normal_subgroup_quotient_mul(s,
            normal_subgroup_quotient_pow(s, normal_subgroup_quotient_project(s, a), n),
            normal_subgroup_quotient_pow(s, normal_subgroup_quotient_project(s, a), m))
} by {
    if is_normal_subgroup(s) {
        normal_subgroup_quotient_project_pow_all(s, a, n + m)
        normal_subgroup_quotient_project_pow_all(s, a, n)
        normal_subgroup_quotient_pow(s, normal_subgroup_quotient_project(s, a), n) =
            normal_subgroup_quotient_project(s, a.pow(n))
        normal_subgroup_quotient_project_pow_all(s, a, m)
        normal_subgroup_quotient_pow(s, normal_subgroup_quotient_project(s, a), m) =
            normal_subgroup_quotient_project(s, a.pow(m))
        pow_add(a, n, m)
        normal_subgroup_quotient_project_mul(s, a.pow(n), a.pow(m))
        normal_subgroup_quotient_pow(s, normal_subgroup_quotient_project(s, a), n + m) =
            normal_subgroup_quotient_mul(s,
                normal_subgroup_quotient_pow(s, normal_subgroup_quotient_project(s, a), n),
                normal_subgroup_quotient_pow(s, normal_subgroup_quotient_project(s, a), m))
    }
}

/// Iterated projected powers multiply exponents: raising the projected `n`-th power to the `m`-th power gives the projected `(n * m)`-th power.
theorem normal_subgroup_quotient_project_pow_pow[G: Group](s: Subgroup[G], a: G, n: Nat, m: Nat) {
    is_normal_subgroup(s) implies
    normal_subgroup_quotient_pow(s,
        normal_subgroup_quotient_pow(s, normal_subgroup_quotient_project(s, a), n), m) =
        normal_subgroup_quotient_pow(s, normal_subgroup_quotient_project(s, a), n * m)
} by {
    if is_normal_subgroup(s) {
        normal_subgroup_quotient_project_pow_all(s, a, n)
        normal_subgroup_quotient_pow(s, normal_subgroup_quotient_project(s, a), n) =
            normal_subgroup_quotient_project(s, a.pow(n))
        normal_subgroup_quotient_project_pow_all(s, a.pow(n), m)
        pow_pow(a, n, m)
        normal_subgroup_quotient_project_pow_all(s, a, n * m)
        normal_subgroup_quotient_pow(s,
            normal_subgroup_quotient_pow(s, normal_subgroup_quotient_project(s, a), n), m) =
            normal_subgroup_quotient_pow(s, normal_subgroup_quotient_project(s, a), n * m)
    }
}

/// Multiplication on the normal-subgroup quotient is associative.
theorem normal_subgroup_quotient_mul_assoc[G: Group](s: Subgroup[G], a: G, b: G, c: G) {
    is_normal_subgroup(s) implies
    normal_subgroup_quotient_mul(s,
        normal_subgroup_quotient_mul(s,
            quotient_over_mk(normal_subgroup_quotient_relation(s), a),
            quotient_over_mk(normal_subgroup_quotient_relation(s), b)),
        quotient_over_mk(normal_subgroup_quotient_relation(s), c)) =
    normal_subgroup_quotient_mul(s,
        quotient_over_mk(normal_subgroup_quotient_relation(s), a),
        normal_subgroup_quotient_mul(s,
            quotient_over_mk(normal_subgroup_quotient_relation(s), b),
            quotient_over_mk(normal_subgroup_quotient_relation(s), c)))
} by {
    if is_normal_subgroup(s) {
        normal_subgroup_quotient_mul_projection(s, a, b)
        normal_subgroup_quotient_mul_projection(s, a * b, c)
        normal_subgroup_quotient_mul_projection(s, b, c)
        normal_subgroup_quotient_mul_projection(s, a, b * c)
        quotient_over_mk(normal_subgroup_quotient_relation(s), (a * b) * c) =
            quotient_over_mk(normal_subgroup_quotient_relation(s), a * (b * c))
        normal_subgroup_quotient_mul(s,
            normal_subgroup_quotient_mul(s,
                quotient_over_mk(normal_subgroup_quotient_relation(s), a),
                quotient_over_mk(normal_subgroup_quotient_relation(s), b)),
            quotient_over_mk(normal_subgroup_quotient_relation(s), c)) =
        normal_subgroup_quotient_mul(s,
            quotient_over_mk(normal_subgroup_quotient_relation(s), a),
            normal_subgroup_quotient_mul(s,
                quotient_over_mk(normal_subgroup_quotient_relation(s), b),
                quotient_over_mk(normal_subgroup_quotient_relation(s), c)))
    }
}

/// The quotient identity is a left identity for quotient multiplication.
theorem normal_subgroup_quotient_one_mul[G: Group](s: Subgroup[G], a: G) {
    is_normal_subgroup(s) implies
    normal_subgroup_quotient_mul(s,
        quotient_over_mk(normal_subgroup_quotient_relation(s), G.1),
        quotient_over_mk(normal_subgroup_quotient_relation(s), a)) =
        quotient_over_mk(normal_subgroup_quotient_relation(s), a)
} by {
    if is_normal_subgroup(s) {
        normal_subgroup_quotient_mul_projection(s, G.1, a)
        G.1 * a = a
    }
}

/// The quotient identity is a right identity for quotient multiplication.
theorem normal_subgroup_quotient_mul_one[G: Group](s: Subgroup[G], a: G) {
    is_normal_subgroup(s) implies
    normal_subgroup_quotient_mul(s,
        quotient_over_mk(normal_subgroup_quotient_relation(s), a),
        quotient_over_mk(normal_subgroup_quotient_relation(s), G.1)) =
        quotient_over_mk(normal_subgroup_quotient_relation(s), a)
} by {
    if is_normal_subgroup(s) {
        normal_subgroup_quotient_mul_projection(s, a, G.1)
        a * G.1 = a
    }
}

/// The quotient inverse is a left inverse for quotient multiplication.
theorem normal_subgroup_quotient_inverse_left[G: Group](s: Subgroup[G], a: G) {
    is_normal_subgroup(s) implies
    normal_subgroup_quotient_mul(s,
        normal_subgroup_quotient_inverse(s,
            quotient_over_mk(normal_subgroup_quotient_relation(s), a)),
        quotient_over_mk(normal_subgroup_quotient_relation(s), a)) =
        quotient_over_mk(normal_subgroup_quotient_relation(s), G.1)
} by {
    if is_normal_subgroup(s) {
        normal_subgroup_quotient_inverse_projection(s, a)
        normal_subgroup_quotient_mul_projection(s, a.inverse, a)
        inverse_left(a)
        normal_subgroup_quotient_mul(s,
            normal_subgroup_quotient_inverse(s, quotient_over_mk(normal_subgroup_quotient_relation(s), a)),
            quotient_over_mk(normal_subgroup_quotient_relation(s), a)) =
            quotient_over_mk(normal_subgroup_quotient_relation(s), G.1)
    }
}

/// The quotient inverse is a right inverse for quotient multiplication.
theorem normal_subgroup_quotient_inverse_right[G: Group](s: Subgroup[G], a: G) {
    is_normal_subgroup(s) implies
    normal_subgroup_quotient_mul(s,
        quotient_over_mk(normal_subgroup_quotient_relation(s), a),
        normal_subgroup_quotient_inverse(s,
            quotient_over_mk(normal_subgroup_quotient_relation(s), a))) =
        quotient_over_mk(normal_subgroup_quotient_relation(s), G.1)
} by {
    if is_normal_subgroup(s) {
        normal_subgroup_quotient_inverse_projection(s, a)
        normal_subgroup_quotient_mul_projection(s, a, a.inverse)
        normal_subgroup_quotient_mul(s,
            quotient_over_mk(normal_subgroup_quotient_relation(s), a),
            normal_subgroup_quotient_inverse(s, quotient_over_mk(normal_subgroup_quotient_relation(s), a))) =
            quotient_over_mk(normal_subgroup_quotient_relation(s), G.1)
    }
}

/// The inverse of a quotient product is the product of the inverse representatives in reverse order.
theorem normal_subgroup_quotient_inverse_mul[G: Group](s: Subgroup[G], a: G, b: G) {
    is_normal_subgroup(s) implies
    normal_subgroup_quotient_inverse(s,
        normal_subgroup_quotient_mul(s,
            quotient_over_mk(normal_subgroup_quotient_relation(s), a),
            quotient_over_mk(normal_subgroup_quotient_relation(s), b))) =
    normal_subgroup_quotient_mul(s,
        normal_subgroup_quotient_inverse(s,
            quotient_over_mk(normal_subgroup_quotient_relation(s), b)),
        normal_subgroup_quotient_inverse(s,
            quotient_over_mk(normal_subgroup_quotient_relation(s), a)))
} by {
    if is_normal_subgroup(s) {
        normal_subgroup_quotient_mul_projection(s, a, b)
        normal_subgroup_quotient_inverse_projection(s, a * b)

        normal_subgroup_quotient_inverse_projection(s, b)
        normal_subgroup_quotient_inverse_projection(s, a)
        normal_subgroup_quotient_mul_projection(s, b.inverse, a.inverse)
        inverse_mul(a, b)
        normal_subgroup_quotient_inverse(s,
            normal_subgroup_quotient_mul(s,
                quotient_over_mk(normal_subgroup_quotient_relation(s), a),
                quotient_over_mk(normal_subgroup_quotient_relation(s), b))) =
        normal_subgroup_quotient_mul(s,
            normal_subgroup_quotient_inverse(s,
                quotient_over_mk(normal_subgroup_quotient_relation(s), b)),
            normal_subgroup_quotient_inverse(s,
                quotient_over_mk(normal_subgroup_quotient_relation(s), a)))
    }
}

/// Multiplication by a quotient inverse cancels a matching left factor.
theorem normal_subgroup_quotient_inverse_mul_cancel_left[G: Group](
    s: Subgroup[G], a: G, b: G
) {
    is_normal_subgroup(s) implies
    normal_subgroup_quotient_mul(s,
        normal_subgroup_quotient_inverse(s,
            quotient_over_mk(normal_subgroup_quotient_relation(s), a)),
        normal_subgroup_quotient_mul(s,
            quotient_over_mk(normal_subgroup_quotient_relation(s), a),
            quotient_over_mk(normal_subgroup_quotient_relation(s), b))) =
        quotient_over_mk(normal_subgroup_quotient_relation(s), b)
} by {
    if is_normal_subgroup(s) {
        normal_subgroup_quotient_mul_projection(s, a, b)
        normal_subgroup_quotient_inverse_projection(s, a)
        normal_subgroup_quotient_mul_projection(s, a.inverse, a * b)
        a.inverse * a * b = G.1 * b
        quotient_over_mk(normal_subgroup_quotient_relation(s), a.inverse * (a * b)) =
            quotient_over_mk(normal_subgroup_quotient_relation(s), b)
        normal_subgroup_quotient_mul(s,
            normal_subgroup_quotient_inverse(s,
                quotient_over_mk(normal_subgroup_quotient_relation(s), a)),
            normal_subgroup_quotient_mul(s,
                quotient_over_mk(normal_subgroup_quotient_relation(s), a),
                quotient_over_mk(normal_subgroup_quotient_relation(s), b))) =
            quotient_over_mk(normal_subgroup_quotient_relation(s), b)
    }
}

/// Multiplication by a quotient inverse cancels a matching right factor.
theorem normal_subgroup_quotient_mul_inverse_cancel_right[G: Group](
    s: Subgroup[G], a: G, b: G
) {
    is_normal_subgroup(s) implies
    normal_subgroup_quotient_mul(s,
        normal_subgroup_quotient_mul(s,
            quotient_over_mk(normal_subgroup_quotient_relation(s), b),
            quotient_over_mk(normal_subgroup_quotient_relation(s), a)),
        normal_subgroup_quotient_inverse(s,
            quotient_over_mk(normal_subgroup_quotient_relation(s), a))) =
        quotient_over_mk(normal_subgroup_quotient_relation(s), b)
} by {
    if is_normal_subgroup(s) {
        normal_subgroup_quotient_mul_projection(s, b, a)
        normal_subgroup_quotient_inverse_projection(s, a)
        normal_subgroup_quotient_mul_projection(s, b * a, a.inverse)
        b * (a * a.inverse) = b * G.1
        quotient_over_mk(normal_subgroup_quotient_relation(s), (b * a) * a.inverse) =
            quotient_over_mk(normal_subgroup_quotient_relation(s), b)
        normal_subgroup_quotient_mul(s,
            normal_subgroup_quotient_mul(s,
                quotient_over_mk(normal_subgroup_quotient_relation(s), b),
                quotient_over_mk(normal_subgroup_quotient_relation(s), a)),
            normal_subgroup_quotient_inverse(s,
                quotient_over_mk(normal_subgroup_quotient_relation(s), a))) =
            quotient_over_mk(normal_subgroup_quotient_relation(s), b)
    }
}

/// Multiplication on a normal-subgroup quotient of a commutative group is commutative.
theorem normal_subgroup_quotient_mul_comm[G: CommGroup](s: Subgroup[G], a: G, b: G) {
    is_normal_subgroup(s) implies
    normal_subgroup_quotient_mul(s,
        quotient_over_mk(normal_subgroup_quotient_relation(s), a),
        quotient_over_mk(normal_subgroup_quotient_relation(s), b)) =
    normal_subgroup_quotient_mul(s,
        quotient_over_mk(normal_subgroup_quotient_relation(s), b),
        quotient_over_mk(normal_subgroup_quotient_relation(s), a))
} by {
    if is_normal_subgroup(s) {
        normal_subgroup_quotient_mul_projection(s, a, b)
        normal_subgroup_quotient_mul_projection(s, b, a)
        normal_subgroup_quotient_mul(s,
            quotient_over_mk(normal_subgroup_quotient_relation(s), a),
            quotient_over_mk(normal_subgroup_quotient_relation(s), b)) =
        normal_subgroup_quotient_mul(s,
            quotient_over_mk(normal_subgroup_quotient_relation(s), b),
            quotient_over_mk(normal_subgroup_quotient_relation(s), a))
    }
}

/// Multiplication of projected representatives is associative.
theorem normal_subgroup_quotient_project_mul_assoc[G: Group](s: Subgroup[G], a: G, b: G, c: G) {
    is_normal_subgroup(s) implies
    normal_subgroup_quotient_mul(s,
        normal_subgroup_quotient_mul(s,
            normal_subgroup_quotient_project(s, a),
            normal_subgroup_quotient_project(s, b)),
        normal_subgroup_quotient_project(s, c)) =
    normal_subgroup_quotient_mul(s,
        normal_subgroup_quotient_project(s, a),
        normal_subgroup_quotient_mul(s,
            normal_subgroup_quotient_project(s, b),
            normal_subgroup_quotient_project(s, c)))
} by {
    if is_normal_subgroup(s) {
        normal_subgroup_quotient_mul_assoc(s, a, b, c)
    }
}

/// The quotient identity is a left identity for projected representatives.
theorem normal_subgroup_quotient_project_one_mul[G: Group](s: Subgroup[G], a: G) {
    is_normal_subgroup(s) implies
    normal_subgroup_quotient_mul(s,
        normal_subgroup_quotient_one(s),
        normal_subgroup_quotient_project(s, a)) =
        normal_subgroup_quotient_project(s, a)
} by {
    if is_normal_subgroup(s) {
        normal_subgroup_quotient_one_mul(s, a)
    }
}

/// The quotient identity is a right identity for projected representatives.
theorem normal_subgroup_quotient_project_mul_one[G: Group](s: Subgroup[G], a: G) {
    is_normal_subgroup(s) implies
    normal_subgroup_quotient_mul(s,
        normal_subgroup_quotient_project(s, a),
        normal_subgroup_quotient_one(s)) =
        normal_subgroup_quotient_project(s, a)
} by {
    if is_normal_subgroup(s) {
        normal_subgroup_quotient_mul_one(s, a)
    }
}

/// Inversion twice fixes a projected representative.
theorem normal_subgroup_quotient_project_inverse_inverse[G: Group](s: Subgroup[G], a: G) {
    is_normal_subgroup(s) implies
    normal_subgroup_quotient_inverse(s,
        normal_subgroup_quotient_inverse(s, normal_subgroup_quotient_project(s, a))) =
        normal_subgroup_quotient_project(s, a)
} by {
    if is_normal_subgroup(s) {
        normal_subgroup_quotient_inverse_inverse(s, a)
    }
}

/// The quotient inverse is a left inverse for projected representatives.
theorem normal_subgroup_quotient_project_inverse_left[G: Group](s: Subgroup[G], a: G) {
    is_normal_subgroup(s) implies
    normal_subgroup_quotient_mul(s,
        normal_subgroup_quotient_inverse(s, normal_subgroup_quotient_project(s, a)),
        normal_subgroup_quotient_project(s, a)) =
        normal_subgroup_quotient_one(s)
} by {
    if is_normal_subgroup(s) {
        normal_subgroup_quotient_inverse_left(s, a)
        normal_subgroup_quotient_one_projection(s)
        normal_subgroup_quotient_mul(s,
            normal_subgroup_quotient_inverse(s, normal_subgroup_quotient_project(s, a)),
            normal_subgroup_quotient_project(s, a)) =
            normal_subgroup_quotient_one(s)
    }
}

/// The quotient inverse is a right inverse for projected representatives.
theorem normal_subgroup_quotient_project_inverse_right[G: Group](s: Subgroup[G], a: G) {
    is_normal_subgroup(s) implies
    normal_subgroup_quotient_mul(s,
        normal_subgroup_quotient_project(s, a),
        normal_subgroup_quotient_inverse(s, normal_subgroup_quotient_project(s, a))) =
        normal_subgroup_quotient_one(s)
} by {
    if is_normal_subgroup(s) {
        normal_subgroup_quotient_inverse_right(s, a)
        normal_subgroup_quotient_one_projection(s)
        normal_subgroup_quotient_mul(s,
            normal_subgroup_quotient_project(s, a),
            normal_subgroup_quotient_inverse(s, normal_subgroup_quotient_project(s, a))) =
            normal_subgroup_quotient_one(s)
    }
}

/// The inverse of a projected product is the reversed product of projected inverses.
theorem normal_subgroup_quotient_project_inverse_mul[G: Group](s: Subgroup[G], a: G, b: G) {
    is_normal_subgroup(s) implies
    normal_subgroup_quotient_inverse(s,
        normal_subgroup_quotient_mul(s,
            normal_subgroup_quotient_project(s, a),
            normal_subgroup_quotient_project(s, b))) =
    normal_subgroup_quotient_mul(s,
        normal_subgroup_quotient_inverse(s, normal_subgroup_quotient_project(s, b)),
        normal_subgroup_quotient_inverse(s, normal_subgroup_quotient_project(s, a)))
} by {
    if is_normal_subgroup(s) {
        normal_subgroup_quotient_inverse_mul(s, a, b)
    }
}

/// A projected quotient inverse cancels a matching left factor.
theorem normal_subgroup_quotient_project_inverse_mul_cancel_left[G: Group](
    s: Subgroup[G], a: G, b: G
) {
    is_normal_subgroup(s) implies
    normal_subgroup_quotient_mul(s,
        normal_subgroup_quotient_inverse(s, normal_subgroup_quotient_project(s, a)),
        normal_subgroup_quotient_mul(s,
            normal_subgroup_quotient_project(s, a),
            normal_subgroup_quotient_project(s, b))) =
        normal_subgroup_quotient_project(s, b)
} by {
    if is_normal_subgroup(s) {
        normal_subgroup_quotient_inverse_mul_cancel_left(s, a, b)
    }
}

/// A projected quotient inverse cancels a matching right factor.
theorem normal_subgroup_quotient_project_mul_inverse_cancel_right[G: Group](
    s: Subgroup[G], a: G, b: G
) {
    is_normal_subgroup(s) implies
    normal_subgroup_quotient_mul(s,
        normal_subgroup_quotient_mul(s,
            normal_subgroup_quotient_project(s, b),
            normal_subgroup_quotient_project(s, a)),
        normal_subgroup_quotient_inverse(s, normal_subgroup_quotient_project(s, a))) =
        normal_subgroup_quotient_project(s, b)
} by {
    if is_normal_subgroup(s) {
        normal_subgroup_quotient_mul_inverse_cancel_right(s, a, b)
    }
}

/// Equal quotient products with a common projected left factor have equal right factors.
theorem normal_subgroup_quotient_project_mul_left_cancel[G: Group](
    s: Subgroup[G], a: G, b: G, c: G
) {
    is_normal_subgroup(s) and
    normal_subgroup_quotient_mul(s,
        normal_subgroup_quotient_project(s, a),
        normal_subgroup_quotient_project(s, b)) =
    normal_subgroup_quotient_mul(s,
        normal_subgroup_quotient_project(s, a),
        normal_subgroup_quotient_project(s, c)) implies
    normal_subgroup_quotient_project(s, b) = normal_subgroup_quotient_project(s, c)
} by {
    if is_normal_subgroup(s) and
    normal_subgroup_quotient_mul(s,
        normal_subgroup_quotient_project(s, a),
        normal_subgroup_quotient_project(s, b)) =
    normal_subgroup_quotient_mul(s,
        normal_subgroup_quotient_project(s, a),
        normal_subgroup_quotient_project(s, c)) {
        normal_subgroup_quotient_project_inverse_mul_cancel_left(s, a, b)
        normal_subgroup_quotient_project_inverse_mul_cancel_left(s, a, c)
        normal_subgroup_quotient_project(s, b) = normal_subgroup_quotient_project(s, c)
    }
}

/// Equal quotient products with a common projected right factor have equal left factors.
theorem normal_subgroup_quotient_project_mul_right_cancel[G: Group](
    s: Subgroup[G], a: G, b: G, c: G
) {
    is_normal_subgroup(s) and
    normal_subgroup_quotient_mul(s,
        normal_subgroup_quotient_project(s, b),
        normal_subgroup_quotient_project(s, a)) =
    normal_subgroup_quotient_mul(s,
        normal_subgroup_quotient_project(s, c),
        normal_subgroup_quotient_project(s, a)) implies
    normal_subgroup_quotient_project(s, b) = normal_subgroup_quotient_project(s, c)
} by {
    if is_normal_subgroup(s) and
    normal_subgroup_quotient_mul(s,
        normal_subgroup_quotient_project(s, b),
        normal_subgroup_quotient_project(s, a)) =
    normal_subgroup_quotient_mul(s,
        normal_subgroup_quotient_project(s, c),
        normal_subgroup_quotient_project(s, a)) {
        normal_subgroup_quotient_project_mul_inverse_cancel_right(s, a, b)
        normal_subgroup_quotient_project_mul_inverse_cancel_right(s, a, c)
        normal_subgroup_quotient_project(s, b) = normal_subgroup_quotient_project(s, c)
    }
}

/// Equality after multiplying projected representatives on the left is exactly equality before multiplying.
theorem normal_subgroup_quotient_project_mul_left_cancel_iff[G: Group](
    s: Subgroup[G], a: G, b: G, c: G
) {
    is_normal_subgroup(s) implies
    (normal_subgroup_quotient_mul(s,
        normal_subgroup_quotient_project(s, a),
        normal_subgroup_quotient_project(s, b)) =
    normal_subgroup_quotient_mul(s,
        normal_subgroup_quotient_project(s, a),
        normal_subgroup_quotient_project(s, c))) =
    (normal_subgroup_quotient_project(s, b) = normal_subgroup_quotient_project(s, c))
} by {
    if is_normal_subgroup(s) {
        if normal_subgroup_quotient_mul(s,
            normal_subgroup_quotient_project(s, a),
            normal_subgroup_quotient_project(s, b)) =
        normal_subgroup_quotient_mul(s,
            normal_subgroup_quotient_project(s, a),
            normal_subgroup_quotient_project(s, c)) {
            normal_subgroup_quotient_project_mul_left_cancel(s, a, b, c)
        }
        if normal_subgroup_quotient_project(s, b) = normal_subgroup_quotient_project(s, c) {
            normal_subgroup_quotient_mul(s,
                normal_subgroup_quotient_project(s, a),
                normal_subgroup_quotient_project(s, b)) =
            normal_subgroup_quotient_mul(s,
                normal_subgroup_quotient_project(s, a),
                normal_subgroup_quotient_project(s, c))
        }
    }
}

/// Equality after multiplying projected representatives on the right is exactly equality before multiplying.
theorem normal_subgroup_quotient_project_mul_right_cancel_iff[G: Group](
    s: Subgroup[G], a: G, b: G, c: G
) {
    is_normal_subgroup(s) implies
    (normal_subgroup_quotient_mul(s,
        normal_subgroup_quotient_project(s, b),
        normal_subgroup_quotient_project(s, a)) =
    normal_subgroup_quotient_mul(s,
        normal_subgroup_quotient_project(s, c),
        normal_subgroup_quotient_project(s, a))) =
    (normal_subgroup_quotient_project(s, b) = normal_subgroup_quotient_project(s, c))
} by {
    if is_normal_subgroup(s) {
        if normal_subgroup_quotient_mul(s,
            normal_subgroup_quotient_project(s, b),
            normal_subgroup_quotient_project(s, a)) =
        normal_subgroup_quotient_mul(s,
            normal_subgroup_quotient_project(s, c),
            normal_subgroup_quotient_project(s, a)) {
            normal_subgroup_quotient_project_mul_right_cancel(s, a, b, c)
        }
        if normal_subgroup_quotient_project(s, b) = normal_subgroup_quotient_project(s, c) {
            normal_subgroup_quotient_mul(s,
                normal_subgroup_quotient_project(s, b),
                normal_subgroup_quotient_project(s, a)) =
            normal_subgroup_quotient_mul(s,
                normal_subgroup_quotient_project(s, c),
                normal_subgroup_quotient_project(s, a))
        }
    }
}

/// Equality after multiplying projected representatives on the left is exactly the normal-subgroup relation.
theorem normal_subgroup_quotient_project_mul_left_eq_iff_rel[G: Group](
    s: Subgroup[G], a: G, b: G, c: G
) {
    is_normal_subgroup(s) implies
    (normal_subgroup_quotient_mul(s,
        normal_subgroup_quotient_project(s, a),
        normal_subgroup_quotient_project(s, b)) =
    normal_subgroup_quotient_mul(s,
        normal_subgroup_quotient_project(s, a),
        normal_subgroup_quotient_project(s, c))) =
    normal_subgroup_rel(s, b, c)
} by {
    if is_normal_subgroup(s) {
        normal_subgroup_quotient_project_mul_left_cancel_iff(s, a, b, c)
        normal_subgroup_quotient_project_eq_iff_rel(s, b, c)
        (normal_subgroup_quotient_mul(s,
            normal_subgroup_quotient_project(s, a),
            normal_subgroup_quotient_project(s, b)) =
        normal_subgroup_quotient_mul(s,
            normal_subgroup_quotient_project(s, a),
            normal_subgroup_quotient_project(s, c))) =
        normal_subgroup_rel(s, b, c)
    }
}

/// Equality after multiplying projected representatives on the right is exactly the normal-subgroup relation.
theorem normal_subgroup_quotient_project_mul_right_eq_iff_rel[G: Group](
    s: Subgroup[G], a: G, b: G, c: G
) {
    is_normal_subgroup(s) implies
    (normal_subgroup_quotient_mul(s,
        normal_subgroup_quotient_project(s, b),
        normal_subgroup_quotient_project(s, a)) =
    normal_subgroup_quotient_mul(s,
        normal_subgroup_quotient_project(s, c),
        normal_subgroup_quotient_project(s, a))) =
    normal_subgroup_rel(s, b, c)
} by {
    if is_normal_subgroup(s) {
        normal_subgroup_quotient_project_mul_right_cancel_iff(s, a, b, c)
        normal_subgroup_quotient_project_eq_iff_rel(s, b, c)
        (normal_subgroup_quotient_mul(s,
            normal_subgroup_quotient_project(s, b),
            normal_subgroup_quotient_project(s, a)) =
        normal_subgroup_quotient_mul(s,
            normal_subgroup_quotient_project(s, c),
            normal_subgroup_quotient_project(s, a))) =
        normal_subgroup_rel(s, b, c)
    }
}

/// Multiplication of projected representatives is commutative in a commutative group quotient.
theorem normal_subgroup_quotient_project_mul_comm[G: CommGroup](s: Subgroup[G], a: G, b: G) {
    is_normal_subgroup(s) implies
    normal_subgroup_quotient_mul(s,
        normal_subgroup_quotient_project(s, a),
        normal_subgroup_quotient_project(s, b)) =
    normal_subgroup_quotient_mul(s,
        normal_subgroup_quotient_project(s, b),
        normal_subgroup_quotient_project(s, a))
} by {
    if is_normal_subgroup(s) {
        normal_subgroup_quotient_mul_comm(s, a, b)
    }
}
