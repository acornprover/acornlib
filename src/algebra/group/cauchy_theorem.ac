/// Cauchy's theorem: enumeration and counting of the tuple space, the orbit
/// partition, and the final fixed-point argument.  The rotation machinery and
/// the orbit-size analysis live in `algebra.group.cauchy`.

from algebra.group import Group, inverse_left, inverse_inverse, right_cancel, left_cancel
from algebra.group.group_action import list_prod, list_prod_nil, list_prod_cons, list_prod_const,
    const_list, rotate, rotate_cons, rotate_length, prime_power_period_implies_order,
    prime_divisor_one_or_self
from algebra.group.cauchy_disjoint import prepend_all, prepend_all_unfold, prepend_all_contains_witness, prepend_all_pairwise_disjoint
from algebra.group.cauchy_unique import flatten_list, all_lists, flatten_list_contains_witness
from algebra.group.cauchy import rotate_iter, rotate_iter_add, rotate_iter_zero, rotate_iter_suc,
    rotate_iter_length, rotate_p_id, rotate_fixed_imp_const, rotate_fixed_imp_exists_const,
    const_rotate_fixed, const_list_append, const_list_unique_pos, const_list_unique_suc,
    period_lt_prime_imp_fixed, rotation_orbit_list, rotation_orbit_set,
    rotation_orbit_list_length_1_or_p, rotation_orbit_list_eq_singleton,
    rotation_orbit_list_length_eq_p, range_map_unique, map_pointwise_eq, map_range_const,
    unique_cons_of_contains, all_iterates_fixed, const_fn, const_fn_apply
from finite_group import FiniteGroup
from nat import Nat, mod_lt, div_mod_decomp, mod_of_decomp, mod_lte,
    pos_of_ne_zero, add_imp_sub, add_imp_sub_left, add_sub, add_one_right, add_one_left, suc_sub_one,
    lt_suc_right, lt_suc, not_lt_zero, trichotomy, lte_imp_not_lt, lt_imp_lte_suc, lte_and_lt,
    lte_antisymm, mul_to_one, divides_mul, gcd_of_prime, pow_add, pow_one, pow_distrib_mul,
    mul_suc_right, mul_comm, mul_zero_right, lte_mul, div_sub_mod, sub_zero,
    has_min, is_min, is_min_apply, is_min_false_below, false_below, false_below_apply,
    add_to_zero, lt_add_left, sub_pos, lte_add_left, lt_imp_lt_suc, zero_or_suc, lt_trans, mul_to_zero
from list import List, map, sum, sum_map_of_pointwise, map_map, map_contains,
    map_contains_of_contains, map_length,
    contains_imp_unique_contains, unique_contains_imp_contains, unique_preserves_contains,
    filter_equivalent_to_and, filter_preserves_unique, list_extensionality,
    add_contains_left, add_contains_right, add_length, range_contains_of_lt, length_range,
    map_add, map_range, not_unique_implies_duplicate, duplicate_implies_duplicate_idx, add_contains_or
from data.basic.functions import is_injective_fn
from list import unique_implies_tail_unique, unique_length, cons_unique_of_tail_unique_not_contains
from data.basic.set import Set, set_ext, list_set, list_set_contains_eq, set_eq_universal_of_forall_contains,
    unique_list_set_cardinality_is_length, cardinality_is_well_defined

numerals Nat

// ==== Enumeration of tuples ====


/// A list of length zero is the empty list.
theorem length_zero_imp_nil_local[T](list: List[T]) {
    list.length = Nat.0 implies list = List.nil[T]
} by {
    match list {
        List.nil {
            if list.length = Nat.0 {
                list = List.nil[T]
            }
        }
        List.cons(head, tail) {
            tail.length.suc != Nat.0
            if list.length = Nat.0 {
                false
            }
        }
    }
}

/// Flattening a cons list of lists concatenates the head list.
theorem flatten_list_cons[T](xs: List[T], rest: List[List[T]]) {
    flatten_list(List.cons(xs, rest)) = xs + flatten_list(rest)
} by {
}



/// Every list in `all_lists` has the prescribed length.
theorem all_lists_member_length[T](items: List[T], n: Nat, t: List[T]) {
    all_lists(items, n).contains(t) implies t.length = n
} by {
    define p(k: Nat) -> Bool {
        forall(u: List[T]) {
            all_lists(items, k).contains(u) implies u.length = k
        }
    }
    forall(u: List[T]) {
        if all_lists(items, Nat.0).contains(u) {
            all_lists(items, Nat.0) = List.singleton[List[T]](List.nil[T])
            List.singleton[List[T]](List.nil[T]).contains(u)
            u = List.nil[T]
            u.length = Nat.0
        }
    }
    p(Nat.0)
    forall(k: Nat) {
        if p(k) {
            forall(u: List[T]) {
                if all_lists(items, k.suc).contains(u) {
                    flatten_list(map[T, List[List[T]]](items, function(x: T) {
                        prepend_all(x, all_lists(items, k))
                    })).contains(u)
                    flatten_list_contains_witness(map[T, List[List[T]]](items, function(x: T) {
                        prepend_all(x, all_lists(items, k))
                    }), u)
                    exists(xs: List[List[T]]) {
                        map[T, List[List[T]]](items, function(x: T) {
                            prepend_all(x, all_lists(items, k))
                        }).contains(xs) and xs.contains(u)
                    }
                    let xs: List[List[T]] satisfy {
                        map[T, List[List[T]]](items, function(x: T) {
                            prepend_all(x, all_lists(items, k))
                        }).contains(xs) and xs.contains(u)
                    }
                    map_contains(items, function(x: T) {
                        prepend_all(x, all_lists(items, k))
                    }, xs)
                    let x: T satisfy {
                        items.contains(x) and
                        prepend_all(x, all_lists(items, k)) = xs
                    }
                    map_contains(all_lists(items, k), function(tt: List[T]) {
                        List.cons(x, tt)
                    }, u)
                    let t0: List[T] satisfy {
                        all_lists(items, k).contains(t0) and List.cons(x, t0) = u
                    }
                    p(k)
                    t0.length = k
                    u.length = t0.length.suc
                    u.length = k.suc
                }
            }
            p(k.suc)
        }
    }
    forall(k: Nat) {
        p(k) implies p(k.suc)
    }
    p(Nat.0) and forall(k: Nat) {
        p(k) implies p(k.suc)
    }
    Nat.induction(p)
    p(n)
    all_lists(items, n).contains(t) implies t.length = n
}

/// Flattening preserves membership from a sublist.
theorem flatten_list_contains_of_member_contains[T](xss: List[List[T]], xs: List[T], u: T) {
    xss.contains(xs) and xs.contains(u) implies flatten_list(xss).contains(u)
} by {
    define p(xss0: List[List[T]]) -> Bool {
        xss0.contains(xs) and xs.contains(u) implies flatten_list(xss0).contains(u)
    }
    if List.nil[List[T]].contains(xs) and xs.contains(u) {
        List.nil[List[T]].contains(xs)
        false
    }
    p(List.nil[List[T]])
    forall(xs0: List[T], rest: List[List[T]]) {
        if p(rest) {
            if List.cons(xs0, rest).contains(xs) and xs.contains(u) {
                if xs0 = xs {
                    flatten_list(List.cons(xs0, rest)) = xs0 + flatten_list(rest)
                    (xs0 + flatten_list(rest)).contains(u)
                    flatten_list(List.cons(xs0, rest)).contains(u)
                }
                if xs0 != xs {
                    List.cons(xs0, rest).contains(xs)
                    rest.contains(xs)
                    p(rest)
                    flatten_list(rest).contains(u)
                    flatten_list(List.cons(xs0, rest)) = xs0 + flatten_list(rest)
                    (xs0 + flatten_list(rest)).contains(u)
                    flatten_list(List.cons(xs0, rest)).contains(u)
                }
                flatten_list(List.cons(xs0, rest)).contains(u)
            }
            p(List.cons(xs0, rest))
        }
    }
    forall(xs0: List[T], rest: List[List[T]]) {
        p(rest) implies p(List.cons(xs0, rest))
    }
    p(List.nil[List[T]]) and forall(xs0: List[T], rest: List[List[T]]) {
        p(rest) implies p(List.cons(xs0, rest))
    }
    List.induction(p)
    forall(xss0: List[List[T]]) {
        p(xss0)
    }
    p(xss)
    xss.contains(xs) and xs.contains(u) implies flatten_list(xss).contains(u)
}

/// A list of the right length with entries in `items` is in `all_lists`.
theorem all_lists_contains_of_length_entries[T](items: List[T], n: Nat, t: List[T]) {
    t.length = n and (forall(x: T) { t.contains(x) implies items.contains(x) }) implies all_lists(items, n).contains(t)
} by {
    define p(k: Nat) -> Bool {
        forall(u: List[T]) {
            u.length = k and (forall(x: T) { u.contains(x) implies items.contains(x) }) implies all_lists(items, k).contains(u)
        }
    }
    forall(u: List[T]) {
        if u.length = Nat.0 and forall(x: T) { u.contains(x) implies items.contains(x) } {
            u.length = Nat.0
            length_zero_imp_nil_local(u)
            u = List.nil[T]
            all_lists(items, Nat.0) = List.singleton[List[T]](List.nil[T])
            List.singleton[List[T]](List.nil[T]).contains(List.nil[T])
            all_lists(items, Nat.0).contains(u)
        }
    }
    p(Nat.0) = forall(u: List[T]) { u.length = Nat.0 and (forall(x: T) { u.contains(x) implies items.contains(x) }) implies all_lists(items, Nat.0).contains(u) }
    p(Nat.0)
    forall(k: Nat) {
        if p(k) {
            forall(u: List[T]) {
                if u.length = k.suc and forall(x: T) { u.contains(x) implies items.contains(x) } {
                    u.length = k.suc
                    match u {
                        List.nil {
                            u.length = Nat.0
                            u.length = k.suc
                            k.suc = Nat.0
                            k.suc != Nat.0
                            false
                        }
                        List.cons(h, rest) {
                            u.length = rest.length.suc
                            rest.length.suc = k.suc
                            rest.length = k
                            items.contains(h)
                            forall(x0: T) {
                                if rest.contains(x0) {
                                    List.cons(h, rest).contains(x0)
                                    u.contains(x0)
                                    items.contains(x0)
                                }
                            }
                            p(k)
                            rest.length = k and (forall(x0: T) { rest.contains(x0) implies items.contains(x0) }) implies all_lists(items, k).contains(rest)
                            all_lists(items, k).contains(rest)
                            map_contains_of_contains(all_lists(items, k), function(tt: List[T]) {
                                List.cons(h, tt)
                            }, rest)
                            map[List[T], List[T]](all_lists(items, k), function(tt: List[T]) {
                                List.cons(h, tt)
                            }).contains(List.cons(h, rest))
                            map_contains_of_contains(items, function(x: T) {
                                prepend_all(x, all_lists(items, k))
                            }, h)
                            map[T, List[List[T]]](items, function(x: T) {
                                prepend_all(x, all_lists(items, k))
                            }).contains(map[List[T], List[T]](all_lists(items, k), function(tt: List[T]) {
                                List.cons(h, tt)
                            }))
                            flatten_list_contains_of_member_contains(map[T, List[List[T]]](items, function(x: T) {
                                prepend_all(x, all_lists(items, k))
                            }), map[List[T], List[T]](all_lists(items, k), function(tt: List[T]) {
                                List.cons(h, tt)
                            }), List.cons(h, rest))
                            all_lists(items, k.suc).contains(List.cons(h, rest))
                            all_lists(items, k.suc).contains(u)
                        }
                    }
                }
            }
            p(k.suc)
        }
    }
    forall(k: Nat) {
        p(k) implies p(k.suc)
    }
    p(Nat.0) and forall(k: Nat) {
        p(k) implies p(k.suc)
    }
    Nat.induction(p)
    p(n)
    t.length = n and (forall(x: T) { t.contains(x) implies items.contains(x) }) implies all_lists(items, n).contains(t)
}

// ==== The tuple space and its cardinality ====

/// The constant function from `T` to `U` returning `x`.
define const_fn_gen[T, U](x: U) -> T -> U {
    function(k: T) { x }
}

/// The constant function evaluates to its value.
theorem const_fn_gen_apply[T, U](x: U, k: T) {
    const_fn_gen[T, U](x)(k) = x
} by {
}

/// The length function on lists.
define list_len[T](xs: List[T]) -> Nat {
    xs.length
}

/// The length function evaluates to the length.
theorem list_len_apply[T](xs: List[T]) {
    list_len(xs) = xs.length
} by {
}

/// The sum of a constant list of naturals is the constant times the length.
theorem sum_const_mul[T](c: Nat, items: List[T]) {
    sum[Nat](map[T, Nat](items, const_fn_gen[T, Nat](c))) = c * items.length
} by {
    define p(xs: List[T]) -> Bool {
        sum[Nat](map[T, Nat](xs, const_fn_gen[T, Nat](c))) = c * xs.length
    }
    map[T, Nat](List.nil[T], const_fn_gen[T, Nat](c)) = List.nil[Nat]
    sum[Nat](List.nil[Nat]) = Nat.0
    sum[Nat](map[T, Nat](List.nil[T], const_fn_gen[T, Nat](c))) = Nat.0
    c * List.nil[T].length = Nat.0
    p(List.nil[T])
    forall(head: T, tail: List[T]) {
        if p(tail) {
            sum[Nat](map[T, Nat](tail, const_fn_gen[T, Nat](c))) = c * tail.length
            const_fn_gen_apply(c, head)
            map[T, Nat](List.cons(head, tail), const_fn_gen[T, Nat](c)) =
                List.cons(const_fn_gen[T, Nat](c)(head), map[T, Nat](tail, const_fn_gen[T, Nat](c)))
            sum[Nat](List.cons(c, map[T, Nat](tail, const_fn_gen[T, Nat](c)))) =
                c + sum[Nat](map[T, Nat](tail, const_fn_gen[T, Nat](c)))
            sum[Nat](map[T, Nat](List.cons(head, tail), const_fn_gen[T, Nat](c))) = c + c * tail.length
            mul_suc_right(c, tail.length)
            c * tail.length.suc = c + c * tail.length
            List.cons(head, tail).length = tail.length.suc
            c * List.cons(head, tail).length = c + c * tail.length
            sum[Nat](map[T, Nat](List.cons(head, tail), const_fn_gen[T, Nat](c))) =
                c * List.cons(head, tail).length
            p(List.cons(head, tail))
        }
    }
    forall(head: T, tail: List[T]) {
        p(tail) implies p(List.cons(head, tail))
    }
    p(List.nil[T]) and forall(head: T, tail: List[T]) {
        p(tail) implies p(List.cons(head, tail))
    }
    List.induction(p)
    forall(xs: List[T]) {
        p(xs)
    }
    p(items)
    sum[Nat](map[T, Nat](items, const_fn_gen[T, Nat](c))) = c * items.length
}

/// The length of a flattened list of lists is the sum of the sublist lengths.
theorem flatten_list_length[T](xss: List[List[T]]) {
    flatten_list(xss).length = sum[Nat](map[List[T], Nat](xss, list_len[T]))
} by {
    define p(xss0: List[List[T]]) -> Bool {
        flatten_list(xss0).length = sum[Nat](map[List[T], Nat](xss0, list_len[T]))
    }
    flatten_list(List.nil[List[T]]).length = Nat.0
    map[List[T], Nat](List.nil[List[T]], list_len[T]) = List.nil[Nat]
    sum[Nat](List.nil[Nat]) = Nat.0
    sum[Nat](map[List[T], Nat](List.nil[List[T]], list_len[T])) = Nat.0
    p(List.nil[List[T]])
    forall(xs0: List[T], rest: List[List[T]]) {
        if p(rest) {
            flatten_list(List.cons(xs0, rest)) = xs0 + flatten_list(rest)
            (xs0 + flatten_list(rest)).length = xs0.length + flatten_list(rest).length
            flatten_list(List.cons(xs0, rest)).length = xs0.length + flatten_list(rest).length
            flatten_list(rest).length = sum[Nat](map[List[T], Nat](rest, list_len[T]))
            flatten_list(List.cons(xs0, rest)).length =
                xs0.length + sum[Nat](map[List[T], Nat](rest, list_len[T]))
            list_len_apply(xs0)
            list_len(xs0) = xs0.length
            map[List[T], Nat](List.cons(xs0, rest), list_len[T]) =
                List.cons(list_len(xs0), map[List[T], Nat](rest, list_len[T]))
            sum[Nat](List.cons(list_len(xs0), map[List[T], Nat](rest, list_len[T]))) =
                list_len(xs0) + sum[Nat](map[List[T], Nat](rest, list_len[T]))
            flatten_list(List.cons(xs0, rest)).length =
                sum[Nat](map[List[T], Nat](List.cons(xs0, rest), list_len[T]))
            p(List.cons(xs0, rest))
        }
    }
    forall(xs0: List[T], rest: List[List[T]]) {
        p(rest) implies p(List.cons(xs0, rest))
    }
    p(List.nil[List[T]]) and forall(xs0: List[T], rest: List[List[T]]) {
        p(rest) implies p(List.cons(xs0, rest))
    }
    List.induction(p)
    forall(xss0: List[List[T]]) {
        p(xss0)
    }
    p(xss)
    flatten_list(xss).length = sum[Nat](map[List[T], Nat](xss, list_len[T]))
}

/// The lists of length `k` over `items` prepended with `x`.
define cons_all[T](x: T, items: List[T], k: Nat) -> List[List[T]] {
    prepend_all(x, all_lists(items, k))
}

/// The length of the prepended lists of length `k` over `items`.
define cons_all_len[T](x: T, items: List[T], k: Nat) -> Nat {
    list_len(cons_all(x, items, k))
}

/// The prepended-lists map as a named function of `x`.
define cons_all_fn[T](items: List[T], k: Nat) -> T -> List[List[T]] {
    function(x: T) { cons_all(x, items, k) }
}

/// The prepended-lists map applied to a point is the prepended lists at that point.
theorem cons_all_fn_apply[T](x: T, items: List[T], k: Nat) {
    cons_all_fn(items, k)(x) = cons_all(x, items, k)
} by {
}

/// The length of the prepended lists is the number of lists of length `k`.
theorem cons_all_len_eq_all_lists_card[T](x: T, items: List[T], k: Nat) {
    cons_all_len(x, items, k) = all_lists(items, k).length
} by {
    cons_all(x, items, k) = prepend_all(x, all_lists(items, k))
    map_length(all_lists(items, k), function(t: List[T]) {
        List.cons(x, t)
    })
    prepend_all(x, all_lists(items, k)).length = all_lists(items, k).length
    cons_all_len(x, items, k) = list_len(cons_all(x, items, k))
    list_len_apply(cons_all(x, items, k))
    list_len(cons_all(x, items, k)) = cons_all(x, items, k).length
    cons_all_len(x, items, k) = all_lists(items, k).length
}

/// The prepended-lists map as a named function of `x`.
define prepend_all_fn[T](items: List[T], k: Nat) -> T -> List[List[T]] {
    function(x: T) { prepend_all(x, all_lists(items, k)) }
}

/// The prepended-lists map applied to a point is the prepended lists at that point.
theorem prepend_all_fn_apply[T](x: T, items: List[T], k: Nat) {
    prepend_all_fn(items, k)(x) = prepend_all(x, all_lists(items, k))
} by {
}

/// Prepending preserves the number of lists.
theorem prepend_all_length[T](x: T, ts: List[List[T]]) {
    prepend_all(x, ts).length = ts.length
} by {
    prepend_all_unfold(x, ts)
    prepend_all(x, ts) = map[List[T], List[T]](ts, function(t: List[T]) {
        List.cons(x, t)
    })
    map_length(ts, function(t: List[T]) {
        List.cons(x, t)
    })
    map[List[T], List[T]](ts, function(t: List[T]) {
        List.cons(x, t)
    }).length = ts.length
    prepend_all(x, ts).length = ts.length
}

/// The length of the prepended lists, evaluated at the length function.
theorem cons_all_list_len[T](x: T, items: List[T], k: Nat) {
    list_len(cons_all(x, items, k)) = cons_all_len(x, items, k)
} by {
}

/// The length of `f(x)`, as a named function of `x`.
define list_len_comp[T](f: T -> List[List[T]], x: T) -> Nat {
    list_len(f(x))
}

/// The length of the prepended lists of length `k` is the number of such lists.
theorem prepend_all_fn_len_eq_card[T](x: T, items: List[T], k: Nat) {
    list_len_comp(prepend_all_fn(items, k), x) = all_lists(items, k).length
} by {
    list_len_comp(prepend_all_fn(items, k), x) = list_len(prepend_all_fn(items, k)(x))
    prepend_all_fn_apply(x, items, k)
    prepend_all_fn(items, k)(x) = prepend_all(x, all_lists(items, k))
    list_len_comp(prepend_all_fn(items, k), x) = list_len(prepend_all(x, all_lists(items, k)))
    list_len_apply(prepend_all(x, all_lists(items, k)))
    list_len(prepend_all(x, all_lists(items, k))) = prepend_all(x, all_lists(items, k)).length
    prepend_all_length(x, all_lists(items, k))
    prepend_all(x, all_lists(items, k)).length = all_lists(items, k).length
    list_len_comp(prepend_all_fn(items, k), x) = all_lists(items, k).length
}

/// The sum of the mapped lengths equals the sum of the composed lengths.
theorem sum_map_map_length[T](items: List[T], f: T -> List[List[T]]) {
    sum[Nat](map[List[List[T]], Nat](map[T, List[List[T]]](items, f), list_len[List[T]])) =
        sum[Nat](map[T, Nat](items, function(x: T) { list_len_comp(f, x) }))
} by {
    define p(xs: List[T]) -> Bool {
        sum[Nat](map[List[List[T]], Nat](map[T, List[List[T]]](xs, f), list_len[List[T]])) =
            sum[Nat](map[T, Nat](xs, function(x: T) { list_len_comp(f, x) }))
    }
    map[T, List[List[T]]](List.nil[T], f) = List.nil[List[List[T]]]
    map[List[List[T]], Nat](List.nil[List[List[T]]], list_len[List[T]]) = List.nil[Nat]
    sum[Nat](List.nil[Nat]) = Nat.0
    map[T, Nat](List.nil[T], function(x: T) { list_len_comp(f, x) }) = List.nil[Nat]
    sum[Nat](List.nil[Nat]) = Nat.0
    sum[Nat](map[List[List[T]], Nat](map[T, List[List[T]]](List.nil[T], f), list_len[List[T]])) =
        sum[Nat](map[T, Nat](List.nil[T], function(x: T) { list_len_comp(f, x) }))
    p(List.nil[T])
    forall(head: T, tail: List[T]) {
        if p(tail) {
            sum[Nat](map[List[List[T]], Nat](map[T, List[List[T]]](tail, f), list_len[List[T]])) =
                sum[Nat](map[T, Nat](tail, function(x: T) { list_len_comp(f, x) }))
            map[T, List[List[T]]](List.cons(head, tail), f) =
                List.cons(f(head), map[T, List[List[T]]](tail, f))
            map[List[List[T]], Nat](List.cons(f(head), map[T, List[List[T]]](tail, f)), list_len[List[T]]) =
                List.cons(list_len(f(head)), map[List[List[T]], Nat](map[T, List[List[T]]](tail, f), list_len[List[T]]))
            sum[Nat](List.cons(list_len(f(head)), map[List[List[T]], Nat](map[T, List[List[T]]](tail, f), list_len[List[T]]))) =
                list_len(f(head)) + sum[Nat](map[List[List[T]], Nat](map[T, List[List[T]]](tail, f), list_len[List[T]]))
            list_len_comp(f, head) = list_len(f(head))
            map[T, Nat](List.cons(head, tail), function(x: T) { list_len_comp(f, x) }) =
                List.cons(list_len_comp(f, head), map[T, Nat](tail, function(x: T) { list_len_comp(f, x) }))
            sum[Nat](List.cons(list_len_comp(f, head), map[T, Nat](tail, function(x: T) { list_len_comp(f, x) }))) =
                list_len_comp(f, head) + sum[Nat](map[T, Nat](tail, function(x: T) { list_len_comp(f, x) }))
            sum[Nat](map[List[List[T]], Nat](map[T, List[List[T]]](List.cons(head, tail), f), list_len[List[T]])) =
                sum[Nat](map[T, Nat](List.cons(head, tail), function(x: T) { list_len_comp(f, x) }))
            p(List.cons(head, tail))
        }
    }
    forall(head: T, tail: List[T]) {
        p(tail) implies p(List.cons(head, tail))
    }
    p(List.nil[T]) and forall(head: T, tail: List[T]) {
        p(tail) implies p(List.cons(head, tail))
    }
    List.induction(p)
    forall(xs: List[T]) {
        p(xs)
    }
    p(items)
    sum[Nat](map[List[List[T]], Nat](map[T, List[List[T]]](items, f), list_len[List[T]])) =
        sum[Nat](map[T, Nat](items, function(x: T) { list_len_comp(f, x) }))
}

/// The number of lists of length `n` over `items` is `items.length` to the `n`.
theorem all_lists_cardinality[T](items: List[T], n: Nat) {
    all_lists(items, n).length = items.length.pow(n)
} by {
    define p(k: Nat) -> Bool {
        all_lists(items, k).length = items.length.pow(k)
    }
    all_lists(items, Nat.0) = List.singleton[List[T]](List.nil[T])
    List.singleton[List[T]](List.nil[T]) = List.cons(List.nil[T], List.nil[List[T]])
    List.cons(List.nil[T], List.nil[List[T]]).length = List.nil[List[T]].length.suc
    List.nil[List[T]].length = Nat.0
    List.singleton[List[T]](List.nil[T]).length = Nat.1
    items.length.pow(Nat.0) = Nat.1
    p(Nat.0)
    forall(k: Nat) {
        if p(k) {
            all_lists(items, k).length = items.length.pow(k)
            all_lists(items, k.suc) = flatten_list(map[T, List[List[T]]](items, function(x: T) {
                prepend_all(x, all_lists(items, k))
            }))
            forall(x: T) {
                if items.contains(x) {
                    prepend_all_fn_apply(x, items, k)
                    prepend_all_fn(items, k)(x) = prepend_all(x, all_lists(items, k))
                }
            }
            map_pointwise_eq(items, function(x: T) {
                prepend_all(x, all_lists(items, k))
            }, prepend_all_fn(items, k))
            map[T, List[List[T]]](items, function(x: T) {
                prepend_all(x, all_lists(items, k))
            }) = map[T, List[List[T]]](items, prepend_all_fn(items, k))
            all_lists(items, k.suc) = flatten_list(map[T, List[List[T]]](items, prepend_all_fn(items, k)))
            all_lists(items, k.suc).length = flatten_list(map[T, List[List[T]]](items, prepend_all_fn(items, k))).length
            flatten_list_length(map[T, List[List[T]]](items, prepend_all_fn(items, k)))
            flatten_list(map[T, List[List[T]]](items, prepend_all_fn(items, k))).length =
                sum[Nat](map[List[List[T]], Nat](map[T, List[List[T]]](items, prepend_all_fn(items, k)), list_len[List[T]]))
            all_lists(items, k.suc).length =
                sum[Nat](map[List[List[T]], Nat](map[T, List[List[T]]](items, prepend_all_fn(items, k)), list_len[List[T]]))
            sum_map_map_length(items, prepend_all_fn(items, k))
            sum[Nat](map[List[List[T]], Nat](map[T, List[List[T]]](items, prepend_all_fn(items, k)), list_len[List[T]])) =
                sum[Nat](map[T, Nat](items, function(x: T) { list_len_comp(prepend_all_fn(items, k), x) }))
            all_lists(items, k.suc).length =
                sum[Nat](map[T, Nat](items, function(x: T) { list_len_comp(prepend_all_fn(items, k), x) }))
            forall(x: T) {
                if items.contains(x) {
                    prepend_all_fn_len_eq_card(x, items, k)
                    list_len_comp(prepend_all_fn(items, k), x) = all_lists(items, k).length
                    list_len_comp(prepend_all_fn(items, k), x) = items.length.pow(k)
                }
            }
            sum_map_of_pointwise(items, function(x: T) { list_len_comp(prepend_all_fn(items, k), x) }, const_fn_gen[T, Nat](items.length.pow(k)))
            sum[Nat](map[T, Nat](items, function(x: T) { list_len_comp(prepend_all_fn(items, k), x) })) =
                sum[Nat](map[T, Nat](items, const_fn_gen[T, Nat](items.length.pow(k))))
            sum_const_mul(items.length.pow(k), items)
            sum[Nat](map[T, Nat](items, const_fn_gen[T, Nat](items.length.pow(k)))) =
                items.length.pow(k) * items.length
            all_lists(items, k.suc).length = items.length.pow(k) * items.length
            pow_add(items.length, k, Nat.1)
            items.length.pow(k) * items.length.pow(Nat.1) = items.length.pow(k + Nat.1)
            items.length.pow(Nat.1) = items.length
            items.length.pow(k) * items.length = items.length.pow(k + Nat.1)
            add_one_right(k)
            k + Nat.1 = k.suc
            all_lists(items, k.suc).length = items.length.pow(k.suc)
            p(k.suc)
        }
    }
    forall(k: Nat) {
        p(k) implies p(k.suc)
    }
    p(Nat.0) and forall(k: Nat) {
        p(k) implies p(k.suc)
    }
    Nat.induction(p)
    p(n)
    all_lists(items, n).length = items.length.pow(n)
}

// ==== The tuple space of product-one tuples ====

/// The tuple obtained by prepending the inverse of the product of `u` to `u`.
define x_tuple[G: Group](u: List[G]) -> List[G] {
    List.cons(list_prod(u).inverse, u)
}

/// The `p`-tuples with product the identity, enumerated by their tails.
define x_list[G: FiniteGroup](p: Nat) -> List[List[G]] {
    map[List[G], List[G]](all_lists(G.elements, p - Nat.1), x_tuple[G])
}

/// The set of `p`-tuples with product the identity.
define x_set[G: FiniteGroup](p: Nat) -> Set[List[G]] {
    list_set(x_list[G](p))
}

/// The length of a tuple with a prepended inverse is one more than its tail.
theorem x_tuple_length[G: Group](u: List[G]) {
    x_tuple(u).length = u.length + Nat.1
} by {
    x_tuple(u) = List.cons(list_prod(u).inverse, u)
    List.cons(list_prod(u).inverse, u).length = u.length.suc
    add_one_right(u.length)
    u.length + Nat.1 = u.length.suc
    x_tuple(u).length = u.length + Nat.1
}

/// The product of a tuple with a prepended inverse is the identity.
theorem x_tuple_prod_one[G: Group](u: List[G]) {
    list_prod(x_tuple(u)) = G.1
} by {
    x_tuple(u) = List.cons(list_prod(u).inverse, u)
    list_prod(List.cons(list_prod(u).inverse, u)) = list_prod(u).inverse * list_prod(u)
    list_prod(u).inverse * list_prod(u) = G.1
    list_prod(x_tuple(u)) = G.1
}

/// Every tuple in `x_list` has product the identity.
theorem x_list_member_prod_one[G: FiniteGroup](p: Nat, t: List[G]) {
    x_list[G](p).contains(t) implies list_prod(t) = G.1
} by {
    if x_list[G](p).contains(t) {
        map_contains(all_lists(G.elements, p - Nat.1), x_tuple[G], t)
        let u: List[G] satisfy {
            all_lists(G.elements, p - Nat.1).contains(u) and x_tuple(u) = t
        }
        x_tuple_prod_one(u)
        list_prod(x_tuple(u)) = G.1
        list_prod(t) = G.1
    }
}

/// Every tuple in `x_list` has length `p`.
theorem x_list_member_length[G: FiniteGroup](p: Nat, t: List[G]) {
    p.is_prime and x_list[G](p).contains(t) implies t.length = p
} by {
    if p.is_prime and x_list[G](p).contains(t) {
        map_contains(all_lists(G.elements, p - Nat.1), x_tuple[G], t)
        let u: List[G] satisfy {
            all_lists(G.elements, p - Nat.1).contains(u) and x_tuple(u) = t
        }
        all_lists_member_length(G.elements, p - Nat.1, u)
        u.length = p - Nat.1
        x_tuple_length(u)
        x_tuple(u).length = u.length + Nat.1
        p.is_prime
        Nat.1 < p
        Nat.1 <= p
        add_sub(p, Nat.1)
        p - Nat.1 + Nat.1 = p
        u.length + Nat.1 = p - Nat.1 + Nat.1
        u.length + Nat.1 = p
        t.length = p
    }
}


/// From `a * b = e` the left factor is the inverse of the right.
theorem mul_eq_one_imp_left_inverse[G: Group](a: G, b: G) {
    a * b = G.1 implies a = b.inverse
} by {
    if a * b = G.1 {
        b * b.inverse = G.1
        a * b = b * b.inverse
        right_cancel(b, a, b.inverse)
        a = b.inverse
    }
}

/// Every tuple of length `p` with product the identity is in `x_list`.
theorem x_list_member_of_length_prod_one[G: FiniteGroup](p: Nat, t: List[G]) {
    p.is_prime and t.length = p and list_prod(t) = G.1 implies x_list[G](p).contains(t)
} by {
    if p.is_prime and t.length = p and list_prod(t) = G.1 {
        match t {
            List.nil {
                t.length = Nat.0
                t.length = p
                p = Nat.0
                p.is_prime
                Nat.1 < p
                Nat.1 < Nat.0
                not_lt_zero(Nat.1)
                false
            }
            List.cons(h, rest) {
                list_prod(t) = h * list_prod(rest)
                h * list_prod(rest) = G.1
                mul_eq_one_imp_left_inverse(h, list_prod(rest))
                h = list_prod(rest).inverse
                x_tuple(rest) = List.cons(list_prod(rest).inverse, rest)
                x_tuple(rest) = t
                t.length = rest.length.suc
                rest.length.suc = p
                add_one_right(rest.length)
                rest.length + Nat.1 = p
                add_imp_sub(rest.length, Nat.1, p)
                rest.length = p - Nat.1
                forall(x: G) {
                    G.elements.contains(x)
                }
                forall(x: G) {
                    if rest.contains(x) {
                        G.elements.contains(x)
                    }
                }
                all_lists_contains_of_length_entries(G.elements, p - Nat.1, rest)
                rest.length = p - Nat.1 and (forall(x: G) { rest.contains(x) implies G.elements.contains(x) }) implies all_lists(G.elements, p - Nat.1).contains(rest)
                all_lists(G.elements, p - Nat.1).contains(rest)
                map_contains_of_contains(all_lists(G.elements, p - Nat.1), x_tuple[G], rest)
                map[List[G], List[G]](all_lists(G.elements, p - Nat.1), x_tuple[G]).contains(x_tuple(rest))
                x_list[G](p).contains(x_tuple(rest))
                x_list[G](p).contains(t)
            }
        }
    }
}

// ==== Cauchy's theorem (final step, TODO) ====
//
// The machinery above gives all the ingredients of Cauchy's theorem:
//   * `x_list[G](p)` enumerates exactly the `p`-tuples of group elements with
//     product the identity (`x_list_member_prod_one`, `x_list_member_length`,
//     `x_list_member_of_length_prod_one`), and its length is `|G|^(p-1)`
//     (`all_lists_cardinality`), so `p` divides the number of such tuples.
//   * Rotation acts on the tuple space with orbits of size one or `p`
//     (`rotation_orbit_list_length_1_or_p`), so the number of fixed points
//     (the constant tuples `(x, ..., x)` with `x.pow(p) = e`) is congruent to
//     the number of tuples modulo `p`.
//   * `prime_power_period_implies_order` (in `algebra.group.group_action`)
//     upgrades a nontrivial solution of `x.pow(p) = e` to an element of order
//     `p`.
//
// The remaining steps to close the proof are:
//   1. `all_lists_unique`: the enumeration of length-`n` lists over a unique
//      list is duplicate-free (the pairwise-disjointness of the prepended
//      sublists is proved in `algebra.group.cauchy_disjoint`, and
//      `add_unique_of_disjoint_unique` is proved in
//      `algebra.group.cauchy_unique`; the remaining piece is an induction
//      bundling these into uniqueness of the flattened sublist list).
//   2. `x_list_unique` follows from `all_lists_unique` by injectivity of
//      `x_tuple`, giving `|X| = |G|^(p-1)` as a set cardinality.
//   3. The orbit partition of `x_set[G](p)` (orbit representatives, pairwise
//      disjointness, exact cover) gives `|X| = F + p * M` where `F` counts
//      the fixed points.
//   4. The fixed points are in bijection with the solutions of
//      `x.pow(p) = e`, counted by `G.elements.filter(...)`; since
//      `p | |X|`, the count `F` is divisible by `p` and exceeds one, so a
//      nontrivial solution exists, and `prime_power_period_implies_order`
//      completes:
//
//      theorem cauchy_theorem[G: FiniteGroup](p: Nat) {
//          p.is_prime and p.divides(G.order) implies exists(g: G) {
//              g != G.1 and g.pow(p) = G.1
//          }
//      }
