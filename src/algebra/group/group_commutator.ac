/// The group commutator element.
///
/// This file contains the first definitional slice of Mathlib's
/// `Mathlib.Algebra.Group.Commutator` target: the multiplicative commutator
/// element and its defining expansion.

from algebra.group import Group

/// The commutator element of two elements of a group.
define commutator_element[G: Group](g: G, h: G) -> G {
    g * h * g.inverse * h.inverse
}

/// The commutator element is the product `g * h * g.inverse * h.inverse`.
theorem commutator_element_def[G: Group](g: G, h: G) {
    commutator_element(g, h) = g * h * g.inverse * h.inverse
}
