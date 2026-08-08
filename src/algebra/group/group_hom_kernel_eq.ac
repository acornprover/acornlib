/// Small group-hom kernel equality helpers.

from algebra.group import Group, GroupHom, group_hom_mul, group_hom_inv, right_cancel

/// If the image of `a * b.inverse` is the identity, then `a` and `b` have
/// equal homomorphic images.
theorem group_hom_eq_of_mul_inverse_maps_one[G: Group, H: Group](
    f: GroupHom[G, H], a: G, b: G
) {
    f.hom(a * b.inverse) = H.1 implies f.hom(a) = f.hom(b)
} by {
    if f.hom(a * b.inverse) = H.1 {
        group_hom_mul(f, a, b.inverse)
        group_hom_inv(f, b)
        f.hom(a) * f.hom(b).inverse = f.hom(b) * f.hom(b).inverse
        right_cancel(f.hom(b).inverse, f.hom(a), f.hom(b))
        f.hom(a) = f.hom(b)
    }
}
