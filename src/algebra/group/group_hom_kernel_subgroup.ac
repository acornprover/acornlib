/// Compact bridge facts for the bundled subgroup kernel of a group homomorphism.

from algebra.group import Group, GroupHom, group_hom_mul, group_hom_inv
from algebra.subgroup import Subgroup, subgroup_subset, subgroup_subset_antisymm,
    identity_subgroup, identity_subgroup_contains_eq, identity_subgroup_subset,
    group_hom_kernel, group_hom_kernel_contains_eq
from algebra.group.normal_subgroup import is_normal_subgroup
from algebra.group.normal_subgroup_kernel_containment import subgroup_maps_to_one_raw
from algebra.group.normal_subgroup_quotient_hom import normal_subgroup_maps_to_one,
    normal_subgroup_maps_to_one_at
from algebra.hom_kernel_injective import is_trivial_group_hom_kernel,
    group_hom_injective_iff_trivial_kernel
from data.basic.functions import is_injective_fn

/// The kernel subgroup of a group homomorphism is normal.
theorem group_hom_kernel_is_normal_subgroup[G: Group, H: Group](f: GroupHom[G, H]) {
    is_normal_subgroup(group_hom_kernel(f))
} by {
    forall(g: G, n: G) {
        if group_hom_kernel(f).contains(n) {
            group_hom_kernel_contains_eq(f, n)
            f.hom(n) = H.1
            group_hom_mul(f, g, n)
            group_hom_mul(f, g * n, g.inverse)
            group_hom_inv(f, g)
            f.hom(g * n * g.inverse) = f.hom(g * n) * f.hom(g.inverse)
            f.hom(g * n) = f.hom(g) * f.hom(n)
            f.hom(g.inverse) = f.hom(g).inverse
            f.hom(g * n * g.inverse) = f.hom(g) * H.1 * f.hom(g).inverse
            f.hom(g) * H.1 = f.hom(g)
            f.hom(g) * f.hom(g).inverse = H.1
            f.hom(g * n * g.inverse) = H.1
            group_hom_kernel_contains_eq(f, g * n * g.inverse)
            group_hom_kernel(f).contains(g * n * g.inverse)
        }
    }
}

/// A subgroup maps to the codomain identity exactly when it is contained in the
/// bundled kernel subgroup.
theorem normal_subgroup_maps_to_one_iff_subgroup_subset_group_hom_kernel[G: Group, H: Group](
    f: GroupHom[G, H], n: Subgroup[G]
) {
    normal_subgroup_maps_to_one(f, n) = subgroup_subset(n, group_hom_kernel(f))
} by {
    if normal_subgroup_maps_to_one(f, n) {
        forall(a: G) {
            if n.contains(a) {
                normal_subgroup_maps_to_one_at(f, n, a)
                f.hom(a) = H.1
                group_hom_kernel_contains_eq(f, a)
                group_hom_kernel(f).contains(a)
            }
        }
        subgroup_subset(n, group_hom_kernel(f))
    }
    if subgroup_subset(n, group_hom_kernel(f)) {
        forall(a: G) {
            if n.contains(a) {
                subgroup_subset(n, group_hom_kernel(f)) = forall(x: G) {
                    n.contains(x) implies group_hom_kernel(f).contains(x)
                }
                group_hom_kernel(f).contains(a)
                group_hom_kernel_contains_eq(f, a)
                f.hom(a) = H.1
            }
        }
        subgroup_maps_to_one_raw(f.hom, n) = forall(a: G) {
            n.contains(a) implies f.hom(a) = H.1
        }
        subgroup_maps_to_one_raw(f.hom, n)
        normal_subgroup_maps_to_one(f, n) = subgroup_maps_to_one_raw(f.hom, n)
        normal_subgroup_maps_to_one(f, n)
    }
}

/// The bundled kernel subgroup is the identity subgroup exactly when the kernel
/// is trivial elementwise.
theorem group_hom_kernel_eq_identity_subgroup_iff_trivial_kernel[G: Group, H: Group](
    f: GroupHom[G, H]
) {
    (group_hom_kernel(f) = identity_subgroup[G]) = is_trivial_group_hom_kernel(f)
} by {
    if group_hom_kernel(f) = identity_subgroup[G] {
        forall(a: G) {
            if f.hom(a) = H.1 {
                group_hom_kernel_contains_eq(f, a)
                group_hom_kernel(f).contains(a)
                identity_subgroup[G].contains(a)
                identity_subgroup_contains_eq(a)
                a = G.1
            }
        }
        is_trivial_group_hom_kernel(f)
    }
    if is_trivial_group_hom_kernel(f) {
        forall(a: G) {
            if group_hom_kernel(f).contains(a) {
                group_hom_kernel_contains_eq(f, a)
                f.hom(a) = H.1
                is_trivial_group_hom_kernel(f) = forall(x: G) {
                    f.hom(x) = H.1 implies x = G.1
                }
                a = G.1
                identity_subgroup_contains_eq(a)
                identity_subgroup[G].contains(a)
            }
        }
        subgroup_subset(group_hom_kernel(f), identity_subgroup[G])
        identity_subgroup_subset(group_hom_kernel(f))
        subgroup_subset(identity_subgroup[G], group_hom_kernel(f))
        subgroup_subset_antisymm(group_hom_kernel(f), identity_subgroup[G])
        group_hom_kernel(f) = identity_subgroup[G]
    }
}

/// A group homomorphism is injective exactly when its bundled kernel subgroup
/// is the identity subgroup.
theorem group_hom_injective_iff_group_hom_kernel_eq_identity_subgroup[G: Group, H: Group](
    f: GroupHom[G, H]
) {
    is_injective_fn(f.hom) = (group_hom_kernel(f) = identity_subgroup[G])
} by {
    group_hom_injective_iff_trivial_kernel(f)
    group_hom_kernel_eq_identity_subgroup_iff_trivial_kernel(f)
    is_injective_fn(f.hom) = is_trivial_group_hom_kernel(f)
    (group_hom_kernel(f) = identity_subgroup[G]) = is_trivial_group_hom_kernel(f)
    is_injective_fn(f.hom) = (group_hom_kernel(f) = identity_subgroup[G])
}
