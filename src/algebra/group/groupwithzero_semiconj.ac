/// Semiconjugation in a multiplicative structure with zero.

from algebra.mul import Mul
from algebra.zero import Zero

/// A multiplicative structure with a zero that is absorbing on both sides.
typeclass M: MulZeroClass extends Mul, Zero {
    /// Zero is a left absorbing element for multiplication.
    zero_mul(a: M) {
        M.0 * a = M.0
    }

    /// Zero is a right absorbing element for multiplication.
    mul_zero(a: M) {
        a * M.0 = M.0
    }
}

/// True if `x` is semiconjugate to `y` by `a`.
define semiconj_by[M: Mul](a: M, x: M, y: M) -> Bool {
    a * x = y * a
}

/// The semiconjugacy predicate unfolds to the underlying multiplication equality.
theorem semiconj_by_eq[M: Mul](a: M, x: M, y: M) {
    semiconj_by(a, x, y) = (a * x = y * a)
}

/// Zero semiconjugates zero to zero on the right.
theorem semiconj_by_zero_right[M: MulZeroClass](a: M) {
    semiconj_by(a, M.0, M.0)
} by {
    a * M.0 = M.0
    M.0 * a = M.0
    a * M.0 = M.0 * a
    semiconj_by(a, M.0, M.0)
}

/// Zero semiconjugates any two elements on the left.
theorem semiconj_by_zero_left[M: MulZeroClass](x: M, y: M) {
    semiconj_by(M.0, x, y)
} by {
    M.0 * x = M.0
    y * M.0 = M.0
    M.0 * x = y * M.0
    semiconj_by(M.0, x, y)
}
