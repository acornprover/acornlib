from algebra.group import Group, GroupHom, compose_group_hom, compose_group_hom_hom
from data.basic.functions import compose
from data.basic.set import Set, set_image, set_preimage, maps_into_set_image, set_image_contains_witness,
    set_image_subset_iff_subset_preimage, set_image_compose, set_preimage_compose,
    set_image_preimage_closure, set_image_preimage_closure_extensive,
    set_image_preimage_closure_monotone, set_image_preimage_closure_idempotent,
    set_preimage_image_kernel, set_preimage_image_kernel_monotone,
    set_preimage_image_kernel_reductive, set_preimage_image_kernel_idempotent,
    is_subset_monotone_map, is_set_extensive_map, is_set_reductive_map,
    is_set_idempotent_map, is_set_closure_operator, is_set_kernel_operator

/// The image of a set under a group homomorphism.
define group_hom_image[G: Group, H: Group](f: GroupHom[G, H], s: Set[G]) -> Set[H] {
    set_image(s, f.hom)
}

/// The preimage of a set under a group homomorphism.
define group_hom_preimage[G: Group, H: Group](f: GroupHom[G, H], s: Set[H]) -> Set[G] {
    set_preimage(f.hom, s)
}

/// The source closure induced by image followed by preimage along a group homomorphism.
define group_hom_image_preimage_closure[G: Group, H: Group](
    f: GroupHom[G, H],
    s: Set[G]
) -> Set[G] {
    set_image_preimage_closure(f.hom, s)
}

/// The target kernel induced by preimage followed by image along a group homomorphism.
define group_hom_preimage_image_kernel[G: Group, H: Group](
    f: GroupHom[G, H],
    s: Set[H]
) -> Set[H] {
    set_preimage_image_kernel(f.hom, s)
}

/// Membership in the image of a group homomorphism has a source witness.
theorem group_hom_image_contains_witness[G: Group, H: Group](
    f: GroupHom[G, H],
    s: Set[G],
    y: H
) {
    group_hom_image(f, s).contains(y) implies exists(x: G) {
        s.contains(x) and y = f.hom(x)
    }
} by {
    set_image_contains_witness(s, f.hom, y)
}

/// A member of a set maps into its image under a group homomorphism.
theorem group_hom_maps_into_image[G: Group, H: Group](
    f: GroupHom[G, H],
    s: Set[G],
    x: G
) {
    s.contains(x) implies group_hom_image(f, s).contains(f.hom(x))
} by {
    if s.contains(x) {
        maps_into_set_image(s, f.hom, x)
        group_hom_image(f, s) = set_image(s, f.hom)
    }
}

/// Image containment is equivalent to source containment in the preimage.
theorem group_hom_image_subset_iff_subset_preimage[G: Group, H: Group](
    f: GroupHom[G, H],
    s: Set[G],
    t: Set[H]
) {
    group_hom_image(f, s).subset(t) = s.subset(group_hom_preimage(f, t))
} by {
    set_image_subset_iff_subset_preimage(s, t, f.hom)
}

/// Image under a composite group homomorphism is iterated image.
theorem group_hom_image_compose[G: Group, H: Group, K: Group](
    f: GroupHom[H, K],
    g: GroupHom[G, H],
    s: Set[G]
) {
    group_hom_image(compose_group_hom(f, g), s) =
        group_hom_image(f, group_hom_image(g, s))
} by {
    compose_group_hom_hom(f, g)
    set_image_compose(s, g.hom, f.hom)
}

/// Preimage under a composite group homomorphism is iterated preimage.
theorem group_hom_preimage_compose[G: Group, H: Group, K: Group](
    f: GroupHom[H, K],
    g: GroupHom[G, H],
    s: Set[K]
) {
    group_hom_preimage(compose_group_hom(f, g), s) =
        group_hom_preimage(g, group_hom_preimage(f, s))
} by {
    compose_group_hom_hom(f, g)
    set_preimage_compose(f.hom, g.hom, s)
}

/// The source closure induced by a group homomorphism contains the original set.
theorem group_hom_image_preimage_closure_extensive[G: Group, H: Group](
    f: GroupHom[G, H],
    s: Set[G]
) {
    s.subset(group_hom_image_preimage_closure(f, s))
} by {
    set_image_preimage_closure_extensive(f.hom, s)
}

/// The source closure induced by a group homomorphism preserves inclusion.
theorem group_hom_image_preimage_closure_monotone[G: Group, H: Group](
    f: GroupHom[G, H]
) {
    is_subset_monotone_map(group_hom_image_preimage_closure(f))
} by {
    forall(s: Set[G], t: Set[G]) {
        if s.subset(t) {
            set_image_preimage_closure_monotone(f.hom, s, t)
            group_hom_image_preimage_closure(f, s).subset(
                group_hom_image_preimage_closure(f, t)
            )
        }
    }
}

/// The source closure induced by a group homomorphism is idempotent.
theorem group_hom_image_preimage_closure_idempotent[G: Group, H: Group](
    f: GroupHom[G, H],
    s: Set[G]
) {
    group_hom_image_preimage_closure(f, group_hom_image_preimage_closure(f, s)) =
        group_hom_image_preimage_closure(f, s)
} by {
    set_image_preimage_closure_idempotent(f.hom, s)
}

/// The source closure induced by a group homomorphism is a set closure operator.
theorem group_hom_image_preimage_closure_operator[G: Group, H: Group](
    f: GroupHom[G, H]
) {
    is_set_closure_operator(group_hom_image_preimage_closure(f))
} by {
    group_hom_image_preimage_closure_monotone(f)
    forall(s: Set[G]) {
        group_hom_image_preimage_closure_extensive(f, s)
        s.subset(group_hom_image_preimage_closure(f, s))
    }
    forall(s: Set[G]) {
        group_hom_image_preimage_closure_idempotent(f, s)
        group_hom_image_preimage_closure(f, group_hom_image_preimage_closure(f, s)) =
            group_hom_image_preimage_closure(f, s)
    }
}

/// The target kernel induced by a group homomorphism lies in the original set.
theorem group_hom_preimage_image_kernel_reductive[G: Group, H: Group](
    f: GroupHom[G, H],
    s: Set[H]
) {
    group_hom_preimage_image_kernel(f, s).subset(s)
} by {
    set_preimage_image_kernel_reductive(f.hom, s)
}

/// The target kernel induced by a group homomorphism preserves inclusion.
theorem group_hom_preimage_image_kernel_monotone[G: Group, H: Group](
    f: GroupHom[G, H]
) {
    is_subset_monotone_map(group_hom_preimage_image_kernel(f))
} by {
    forall(s: Set[H], t: Set[H]) {
        if s.subset(t) {
            set_preimage_image_kernel_monotone(f.hom, s, t)
            group_hom_preimage_image_kernel(f, s).subset(
                group_hom_preimage_image_kernel(f, t)
            )
        }
    }
}

/// The target kernel induced by a group homomorphism is idempotent.
theorem group_hom_preimage_image_kernel_idempotent[G: Group, H: Group](
    f: GroupHom[G, H],
    s: Set[H]
) {
    group_hom_preimage_image_kernel(f, group_hom_preimage_image_kernel(f, s)) =
        group_hom_preimage_image_kernel(f, s)
} by {
    set_preimage_image_kernel_idempotent(f.hom, s)
}

/// The target kernel induced by a group homomorphism is a set kernel operator.
theorem group_hom_preimage_image_kernel_operator[G: Group, H: Group](
    f: GroupHom[G, H]
) {
    is_set_kernel_operator(group_hom_preimage_image_kernel(f))
} by {
    group_hom_preimage_image_kernel_monotone(f)
    forall(s: Set[H]) {
        group_hom_preimage_image_kernel_reductive(f, s)
        group_hom_preimage_image_kernel(f, s).subset(s)
    }
    is_set_reductive_map(group_hom_preimage_image_kernel(f))
    forall(s: Set[H]) {
        group_hom_preimage_image_kernel_idempotent(f, s)
        group_hom_preimage_image_kernel(f, group_hom_preimage_image_kernel(f, s)) =
            group_hom_preimage_image_kernel(f, s)
    }
    is_set_idempotent_map(group_hom_preimage_image_kernel(f))
}
