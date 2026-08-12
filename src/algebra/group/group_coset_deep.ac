// Coset deepening: a bundled notion of left and right cosets of a subgroup,
// membership criteria, the standard coset identities (an element lies in its
// own coset, cosets are closed under multiplying by subgroup elements on the
// appropriate side), equality of cosets from an intersection witness, and
// the link between the kernel of a homomorphism and its cosets.

from algebra.group import Group, GroupHom, inverse_inverse, inverse_mul
from algebra.subgroup import Subgroup, subgroup_contains_identity, subgroup_mul_mem,
    subgroup_inv_mul_mem, group_hom_kernel, group_hom_kernel_contains_eq,
    same_left_coset_raw_reflexive, same_representative_implies_left_coset_membership_eq_raw,
    intersecting_left_cosets_equal_raw
from algebra.hom_kernel_injective import group_hom_eq_iff_inv_mul_one
from data.basic.set import Set

/// Membership in the left coset of a subgroup by an element.
define left_coset_contains[G: Group](s: Subgroup[G], a: G, x: G) -> Bool {
    s.contains(a.inverse * x)
}

/// An element lies in its own left coset.
theorem left_coset_contains_self[G: Group](s: Subgroup[G], a: G) {
    left_coset_contains(s, a, a)
} by {
    left_coset_contains(s, a, a) = s.contains(a.inverse * a)
    same_left_coset_raw_reflexive(s, a)
    s.contains(a.inverse * a)
    left_coset_contains(s, a, a)
}

/// Multiplying a coset representative by a subgroup element stays in the coset.
theorem left_coset_contains_of_mem[G: Group](s: Subgroup[G], a: G, t: G) {
    s.contains(t) implies left_coset_contains(s, a, a * t)
} by {
    if s.contains(t) {
        left_coset_contains(s, a, a * t) = s.contains(a.inverse * (a * t))
        a.inverse * (a * t) = (a.inverse * a) * t
        a.inverse * a = G.1
        (a.inverse * a) * t = G.1 * t
        G.1 * t = t
        s.contains(t)
        s.contains(a.inverse * (a * t))
        left_coset_contains(s, a, a * t)
    }
}

/// Membership in a left coset means being the representative times a subgroup element.
theorem left_coset_contains_iff[G: Group](s: Subgroup[G], a: G, x: G) {
    left_coset_contains(s, a, x) = exists(t: G) {
        s.contains(t) and x = a * t
    }
} by {
    if left_coset_contains(s, a, x) {
        left_coset_contains(s, a, x) = s.contains(a.inverse * x)
        s.contains(a.inverse * x)
        a * (a.inverse * x) = (a * a.inverse) * x
        a * a.inverse = G.1
        (a * a.inverse) * x = G.1 * x
        G.1 * x = x
        a * (a.inverse * x) = x
        exists(t: G) { s.contains(t) and x = a * t }
    }
    if exists(t: G) { s.contains(t) and x = a * t } {
        let t: G satisfy { s.contains(t) and x = a * t }
        a.inverse * x = a.inverse * (a * t)
        a.inverse * (a * t) = (a.inverse * a) * t
        a.inverse * a = G.1
        (a.inverse * a) * t = G.1 * t
        G.1 * t = t
        s.contains(t)
        s.contains(a.inverse * x)
        left_coset_contains(s, a, x) = s.contains(a.inverse * x)
        left_coset_contains(s, a, x)
    }
    left_coset_contains(s, a, x) = exists(t: G) {
        s.contains(t) and x = a * t
    }
}

/// A coset is closed under multiplying by subgroup elements on the right.
theorem left_coset_contains_right_mul[G: Group](s: Subgroup[G], a: G, x: G, t: G) {
    left_coset_contains(s, a, x) and s.contains(t) implies left_coset_contains(s, a, x * t)
} by {
    if left_coset_contains(s, a, x) and s.contains(t) {
        left_coset_contains(s, a, x) = s.contains(a.inverse * x)
        s.contains(a.inverse * x)
        subgroup_mul_mem(s, a.inverse * x, t)
        s.contains((a.inverse * x) * t)
        (a.inverse * x) * t = a.inverse * (x * t)
        s.contains(a.inverse * (x * t))
        left_coset_contains(s, a, x * t) = s.contains(a.inverse * (x * t))
        left_coset_contains(s, a, x * t)
    }
}

/// Equal coset representatives give equal cosets.
theorem left_coset_eq_of_rep[G: Group](s: Subgroup[G], a: G, b: G, x: G) {
    s.contains(a.inverse * b) implies (left_coset_contains(s, a, x) = left_coset_contains(s, b, x))
} by {
    if s.contains(a.inverse * b) {
        same_representative_implies_left_coset_membership_eq_raw(s, a, b, x)
        s.contains(a.inverse * x) = s.contains(b.inverse * x)
        left_coset_contains(s, a, x) = s.contains(a.inverse * x)
        left_coset_contains(s, b, x) = s.contains(b.inverse * x)
        left_coset_contains(s, a, x) = left_coset_contains(s, b, x)
    }
}

/// Intersecting cosets are equal.
theorem left_coset_intersection_eq[G: Group](s: Subgroup[G], a: G, b: G) {
    exists(x: G) {
        left_coset_contains(s, a, x) and left_coset_contains(s, b, x)
    } implies forall(y: G) {
        left_coset_contains(s, a, y) = left_coset_contains(s, b, y)
    }
} by {
    if exists(x: G) {
        left_coset_contains(s, a, x) and left_coset_contains(s, b, x)
    } {
        let x: G satisfy { left_coset_contains(s, a, x) and left_coset_contains(s, b, x) }
        left_coset_contains(s, a, x) = s.contains(a.inverse * x)
        left_coset_contains(s, b, x) = s.contains(b.inverse * x)
        s.contains(a.inverse * x) and s.contains(b.inverse * x)
        intersecting_left_cosets_equal_raw(s, a, b)
        forall(y: G) {
            s.contains(a.inverse * y) = s.contains(b.inverse * y)
            left_coset_contains(s, a, y) = s.contains(a.inverse * y)
            left_coset_contains(s, b, y) = s.contains(b.inverse * y)
            left_coset_contains(s, a, y) = left_coset_contains(s, b, y)
        }
    }
}

/// Membership in the right coset of a subgroup by an element.
define right_coset_contains[G: Group](s: Subgroup[G], a: G, x: G) -> Bool {
    s.contains(x * a.inverse)
}

/// An element lies in its own right coset.
theorem right_coset_contains_self[G: Group](s: Subgroup[G], a: G) {
    right_coset_contains(s, a, a)
} by {
    right_coset_contains(s, a, a) = s.contains(a * a.inverse)
    a * a.inverse = G.1
    subgroup_contains_identity(s)
    s.contains(G.1)
    s.contains(a * a.inverse)
    right_coset_contains(s, a, a)
}

/// Multiplying a right coset by a subgroup element on the right stays in the coset.
theorem right_coset_contains_of_mem[G: Group](s: Subgroup[G], a: G, t: G) {
    s.contains(t) implies right_coset_contains(s, a, t * a)
} by {
    if s.contains(t) {
        right_coset_contains(s, a, t * a) = s.contains((t * a) * a.inverse)
        (t * a) * a.inverse = t * (a * a.inverse)
        a * a.inverse = G.1
        t * (a * a.inverse) = t * G.1
        t * G.1 = t
        s.contains(t)
        s.contains((t * a) * a.inverse)
        right_coset_contains(s, a, t * a)
    }
}

/// A right coset is closed under multiplying by subgroup elements on the left.
theorem right_coset_contains_left_mul[G: Group](s: Subgroup[G], a: G, x: G, t: G) {
    right_coset_contains(s, a, x) and s.contains(t) implies right_coset_contains(s, a, t * x)
} by {
    if right_coset_contains(s, a, x) and s.contains(t) {
        right_coset_contains(s, a, x) = s.contains(x * a.inverse)
        s.contains(x * a.inverse)
        subgroup_mul_mem(s, t, x * a.inverse)
        s.contains(t * (x * a.inverse))
        t * (x * a.inverse) = (t * x) * a.inverse
        s.contains((t * x) * a.inverse)
        right_coset_contains(s, a, t * x) = s.contains((t * x) * a.inverse)
        right_coset_contains(s, a, t * x)
    }
}

/// Two elements have equal images under a homomorphism exactly when the second
/// lies in the left coset of the kernel by the first.
theorem group_hom_eq_iff_same_kernel_coset[G: Group, H: Group](f: GroupHom[G, H], a: G, b: G) {
    (f.hom(a) = f.hom(b)) = left_coset_contains(group_hom_kernel(f), a, b)
} by {
    group_hom_eq_iff_inv_mul_one(f, a, b)
    (f.hom(a) = f.hom(b)) = (f.hom(a.inverse * b) = H.1)
    group_hom_kernel_contains_eq(f, a.inverse * b)
    group_hom_kernel(f).contains(a.inverse * b) = (f.hom(a.inverse * b) = H.1)
    left_coset_contains(group_hom_kernel(f), a, b) = group_hom_kernel(f).contains(a.inverse * b)
    (f.hom(a) = f.hom(b)) = left_coset_contains(group_hom_kernel(f), a, b)
}

/// In a subgroup, `a.inverse * b` lies in the subgroup exactly when `b` does,
/// provided `a` itself lies in the subgroup.
theorem subgroup_inv_mul_mem_iff[G: Group](s: Subgroup[G], a: G, b: G) {
    s.contains(a) implies (s.contains(a.inverse * b) = s.contains(b))
} by {
    if s.contains(a) {
        if s.contains(a.inverse * b) {
            subgroup_mul_mem(s, a, a.inverse * b)
            s.contains(a * (a.inverse * b))
            a * (a.inverse * b) = (a * a.inverse) * b
            a * a.inverse = G.1
            (a * a.inverse) * b = G.1 * b
            G.1 * b = b
            s.contains(b)
        }
        if s.contains(b) {
            subgroup_inv_mul_mem(s, a, b)
            s.contains(a.inverse * b)
        }
        s.contains(a.inverse * b) = s.contains(b)
    }
}

/// A subgroup element lies in the left coset of any element of the subgroup.
theorem left_coset_contains_mem[G: Group](s: Subgroup[G], a: G, x: G) {
    s.contains(x) implies left_coset_contains(s, x, a) = s.contains(a)
} by {
    if s.contains(x) {
        left_coset_contains(s, x, a) = s.contains(x.inverse * a)
        subgroup_inv_mul_mem_iff(s, x, a)
        s.contains(x.inverse * a) = s.contains(a)
        left_coset_contains(s, x, a) = s.contains(a)
    }
}
