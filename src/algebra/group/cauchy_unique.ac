/// Uniqueness of the tuple enumeration, isolated in a small module so that
/// proof search stays fast.  All lists of a fixed length over a list are
/// enumerated without duplicates, and the product-one tuple enumeration of
/// `algebra.group.cauchy_theorem` is therefore duplicate-free.

from list import List, map, map_contains, map_contains_of_contains, map_length,
    unique_implies_tail_unique, unique_length, cons_unique_of_tail_unique_not_contains,
    add_contains_or
from nat import Nat
from algebra.group.cauchy_disjoint import prepend_all, prepend_all_unfold,
    prepend_all_contains_witness, prepend_all_pairwise_disjoint
from data.basic.functions import is_injective_fn

numerals Nat

/// Flattening a list of lists.
define flatten_list[T](xss: List[List[T]]) -> List[T] {
    match xss {
        List.nil {
            List.nil[T]
        }
        List.cons(xs, rest) {
            xs + flatten_list(rest)
        }
    }
}

/// Flattening the empty list of lists is the empty list.
theorem flatten_list_nil[T] {
    flatten_list(List.nil[List[T]]) = List.nil[T]
} by {
}

/// All lists of length `n` whose entries lie in `items`.
define all_lists[T](items: List[T], n: Nat) -> List[List[T]] {
    match n {
        Nat.zero {
            List.singleton[List[T]](List.nil[T])
        }
        Nat.suc(m) {
            flatten_list(map[T, List[List[T]]](items, function(x: T) {
                prepend_all(x, all_lists(items, m))
            }))
        }
    }
}

/// Membership in a flattened list of lists is membership in one of the sublists.
theorem flatten_list_contains_witness[T](xss: List[List[T]], u: T) {
    flatten_list(xss).contains(u) implies exists(xs: List[T]) {
        xss.contains(xs) and xs.contains(u)
    }
} by {
    define p(xss0: List[List[T]]) -> Bool {
        flatten_list(xss0).contains(u) implies exists(xs: List[T]) {
            xss0.contains(xs) and xs.contains(u)
        }
    }
    if flatten_list(List.nil[List[T]]).contains(u) {
        flatten_list(List.nil[List[T]]) = List.nil[T]
        List.nil[T].contains(u)
        false
    }
    p(List.nil[List[T]])
    forall(xs0: List[T], rest: List[List[T]]) {
        if p(rest) {
            if flatten_list(List.cons(xs0, rest)).contains(u) {
                flatten_list(List.cons(xs0, rest)) = xs0 + flatten_list(rest)
                (xs0 + flatten_list(rest)).contains(u)
                if xs0.contains(u) {
                    exists(xs: List[T]) {
                        List.cons(xs0, rest).contains(xs) and xs.contains(u)
                    }
                }
                if flatten_list(rest).contains(u) {
                    p(rest)
                    exists(xs: List[T]) {
                        rest.contains(xs) and xs.contains(u)
                    }
                    let xs: List[T] satisfy {
                        rest.contains(xs) and xs.contains(u)
                    }
                    exists(ys: List[T]) {
                        List.cons(xs0, rest).contains(ys) and ys.contains(u)
                    }
                }
                exists(xs: List[T]) {
                    List.cons(xs0, rest).contains(xs) and xs.contains(u)
                }
            }
            p(List.cons(xs0, rest)) = (flatten_list(List.cons(xs0, rest)).contains(u) implies exists(xs: List[T]) {
                List.cons(xs0, rest).contains(xs) and xs.contains(u)
            })
            p(List.cons(xs0, rest))
        }
    }
    forall(xs0: List[T], rest: List[List[T]]) {
        p(rest) implies p(List.cons(xs0, rest))
    }
    p(List.nil[List[T]]) and forall(xs0: List[T], rest: List[List[T]]) {
        p(rest) implies p(List.cons(xs0, rest))
    }
    List.induction(p)
    forall(xss0: List[List[T]]) {
        p(xss0)
    }
    p(xss)
    flatten_list(xss).contains(u) implies exists(xs: List[T]) {
        xss.contains(xs) and xs.contains(u)
    }
}

/// Prepending a fixed element is injective as a function on lists.
theorem cons_injective_fn[T](x: T) {
    is_injective_fn(function(t: List[T]) { List.cons(x, t) })
} by {
    forall(u: List[T], v: List[T]) {
        if (function(t: List[T]) { List.cons(x, t) })(u) = (function(t: List[T]) { List.cons(x, t) })(v) {
            List.cons(x, u) = List.cons(x, v)
            u = v
        }
    }
}

/// The head of a unique cons list is not in the tail.
theorem unique_cons_not_contains_local[T](head: T, tail: List[T]) {
    List.cons(head, tail).is_unique implies not tail.contains(head)
} by {
    if List.cons(head, tail).is_unique {
        if tail.contains(head) {
            let cons_list = List.cons(head, tail)
            List.cons(head, tail).unique = tail.unique
            tail.unique = List.cons(head, tail)
            unique_length(tail)
            List.cons(head, tail).length = tail.length.suc
            tail.length.suc <= tail.length
            false
        }
        not tail.contains(head)
    }
}


/// Mapping an injective function over a unique list gives a unique list.
theorem map_unique_of_injective[T, U](items: List[T], f: T -> U) {
    items.is_unique and is_injective_fn(f) implies map(items, f).is_unique
} by {
    if items.is_unique and is_injective_fn(f) {
        define p(xs: List[T]) -> Bool {
            xs.is_unique implies map(xs, f).is_unique
        }
        map(List.nil[T], f) = List.nil[U]
        List.nil[U].is_unique
        p(List.nil[T])
        forall(head: T, tail: List[T]) {
            if p(tail) {
                if List.cons(head, tail).is_unique {
                    unique_implies_tail_unique(head, tail)
                    tail.is_unique
                    p(tail)
                    map(tail, f).is_unique
                    List.cons(head, tail).is_unique
                    List.cons(head, tail).unique = List.cons(head, tail)
                    unique_cons_not_contains_local(head, tail)
                    not tail.contains(head)
                    map(tail, f).unique = map(tail, f)
                    if map(tail, f).contains(f(head)) {
                        map_contains(tail, f, f(head))
                        let x: T satisfy { tail.contains(x) and f(x) = f(head) }
                        f(x) = f(head)
                        is_injective_fn(f)
                        x = head
                        tail.contains(head)
                        false
                    }
                    map(List.cons(head, tail), f) = List.cons(f(head), map(tail, f))
                    List.cons(f(head), map(tail, f)).is_unique
                    map(List.cons(head, tail), f).is_unique
                }
                p(List.cons(head, tail))
            }
        }
        forall(head: T, tail: List[T]) {
            p(tail) implies p(List.cons(head, tail))
        }
        p(List.nil[T]) and forall(head: T, tail: List[T]) {
            p(tail) implies p(List.cons(head, tail))
        }
        List.induction(p)
        forall(xs: List[T]) {
            p(xs)
        }
        p(items)
        map(items, f).is_unique
    }
}

/// Concatenating unique disjoint lists gives a unique list.
theorem add_unique_of_disjoint_unique[T](a: List[T], b: List[T]) {
    a.is_unique and b.is_unique and (forall(u: T) { a.contains(u) implies not b.contains(u) })
    implies (a + b).is_unique
} by {
    if a.is_unique and b.is_unique and forall(u: T) { a.contains(u) implies not b.contains(u) } {
        define p(xs: List[T]) -> Bool {
            xs.is_unique and (forall(u: T) { xs.contains(u) implies not b.contains(u) }) implies (xs + b).is_unique
        }
        if List.nil[T].is_unique and forall(u: T) { List.nil[T].contains(u) implies not b.contains(u) } {
            List.nil[T] + b = b
            b.is_unique
            (List.nil[T] + b).is_unique
        }
        p(List.nil[T])
        forall(head: T, tail: List[T]) {
            if p(tail) {
                if List.cons(head, tail).is_unique and
                    forall(u: T) { List.cons(head, tail).contains(u) implies not b.contains(u) } {
                    List.cons(head, tail).is_unique
                    unique_cons_not_contains_local(head, tail)
                    not tail.contains(head)
                    forall(u: T) { tail.contains(u) implies not b.contains(u) }
                    unique_implies_tail_unique(head, tail)
                    tail.is_unique
                    p(tail)
                    (tail + b).is_unique
                    forall(u: T) { List.cons(head, tail).contains(u) implies not b.contains(u) }
                    not b.contains(head)
                    if (tail + b).contains(head) {
                        add_contains_or(tail, b, head)
                        if tail.contains(head) {
                            not tail.contains(head)
                            false
                        }
                        if b.contains(head) {
                            not b.contains(head)
                            false
                        }
                        false
                    }
                    not (tail + b).contains(head)
                    cons_unique_of_tail_unique_not_contains(head, tail + b)
                    (List.cons(head, tail) + b).is_unique
                }
                p(List.cons(head, tail))
            }
        }
        forall(head: T, tail: List[T]) {
            p(tail) implies p(List.cons(head, tail))
        }
        p(List.nil[T]) and forall(head: T, tail: List[T]) {
            p(tail) implies p(List.cons(head, tail))
        }
        List.induction(p)
        forall(xs: List[T]) {
            p(xs)
        }
        p(a)
        a.is_unique and (forall(u: T) { a.contains(u) implies not b.contains(u) }) implies (a + b).is_unique
        (a + b).is_unique
    }
}
