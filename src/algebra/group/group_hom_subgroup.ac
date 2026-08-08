from algebra.group import Group, GroupHom, group_hom_one, group_hom_mul, group_hom_inv
from algebra.subgroup import Subgroup, identity_constraint, closure_constraint, inverse_constraint,
    subgroup_constraint, subgroup_contains_identity, subgroup_mul_mem, subgroup_inv_mem,
    subgroup_subset, subgroup_preimage, subgroup_preimage_contains_eq

/// The image membership predicate of a subgroup under a group homomorphism.
define subgroup_image_contains[G: Group, H: Group](
    f: GroupHom[G, H],
    s: Subgroup[G],
    y: H
) -> Bool {
    exists(x: G) {
        s.contains(x) and f.hom(x) = y
    }
}

/// The image of a subgroup contains the identity.
theorem subgroup_image_identity_constraint[G: Group, H: Group](
    f: GroupHom[G, H],
    s: Subgroup[G]
) {
    identity_constraint(subgroup_image_contains(f, s))
} by {
    subgroup_contains_identity(s)
    group_hom_one(f)
    subgroup_image_contains(f, s, H.1)
}

/// The image of a subgroup is closed under multiplication.
theorem subgroup_image_closure_constraint[G: Group, H: Group](
    f: GroupHom[G, H],
    s: Subgroup[G]
) {
    closure_constraint(subgroup_image_contains(f, s))
} by {
    forall(a: H, b: H) {
        if subgroup_image_contains(f, s, a) and subgroup_image_contains(f, s, b) {
            let x: G satisfy { s.contains(x) and f.hom(x) = a }
            let y: G satisfy { s.contains(y) and f.hom(y) = b }
            subgroup_mul_mem(s, x, y)
            group_hom_mul(f, x, y)
            subgroup_image_contains(f, s, a * b)
        }
    }
}

/// The image of a subgroup is closed under inverses.
theorem subgroup_image_inverse_constraint[G: Group, H: Group](
    f: GroupHom[G, H],
    s: Subgroup[G]
) {
    inverse_constraint(subgroup_image_contains(f, s))
} by {
    forall(a: H) {
        if subgroup_image_contains(f, s, a) {
            let x: G satisfy { s.contains(x) and f.hom(x) = a }
            subgroup_inv_mem(s, x)
            group_hom_inv(f, x)
            exists(z: G) {
                s.contains(z) and f.hom(z) = a.inverse
            }
            subgroup_image_contains(f, s, a.inverse)
        }
    }
}

/// The image of a subgroup is a subgroup.
theorem subgroup_image_constraint[G: Group, H: Group](
    f: GroupHom[G, H],
    s: Subgroup[G]
) {
    subgroup_constraint(subgroup_image_contains(f, s))
} by {
    subgroup_image_identity_constraint(f, s)
    subgroup_image_closure_constraint(f, s)
    subgroup_image_inverse_constraint(f, s)
}

/// The image of a subgroup under a group homomorphism.
let subgroup_image[G: Group, H: Group](f: GroupHom[G, H], s: Subgroup[G]) -> result: Subgroup[H] satisfy {
    Subgroup.new(subgroup_image_contains(f, s)) = Option.some(result)
} by {
    subgroup_image_constraint(f, s)
}

/// Membership in a subgroup image means having a source member mapping to the element.
theorem subgroup_image_contains_eq[G: Group, H: Group](
    f: GroupHom[G, H],
    s: Subgroup[G],
    y: H
) {
    subgroup_image(f, s).contains(y) = exists(x: G) {
        s.contains(x) and f.hom(x) = y
    }
} by {
    subgroup_image(f, s).contains(y) = subgroup_image_contains(f, s, y)
}

/// A member of a source subgroup maps into its subgroup image.
theorem subgroup_image_contains_map[G: Group, H: Group](
    f: GroupHom[G, H],
    s: Subgroup[G],
    x: G
) {
    s.contains(x) implies subgroup_image(f, s).contains(f.hom(x))
} by {
    if s.contains(x) {
        f.hom(x) = f.hom(x)
        subgroup_image_contains_eq(f, s, f.hom(x))
        subgroup_image(f, s).contains(f.hom(x))
    }
}

/// A member of a subgroup image has a source witness in the original subgroup.
theorem subgroup_image_contains_witness[G: Group, H: Group](
    f: GroupHom[G, H],
    s: Subgroup[G],
    y: H
) {
    subgroup_image(f, s).contains(y) implies exists(x: G) {
        s.contains(x) and f.hom(x) = y
    }
} by {
    if subgroup_image(f, s).contains(y) {
        subgroup_image_contains_eq(f, s, y)
        exists(x: G) {
            s.contains(x) and f.hom(x) = y
        }
    }
}

/// Image containment is equivalent to source containment in the preimage.
theorem subgroup_image_subset_iff_subset_preimage[G: Group, H: Group](
    f: GroupHom[G, H],
    s: Subgroup[G],
    t: Subgroup[H]
) {
    subgroup_subset(subgroup_image(f, s), t) = subgroup_subset(s, subgroup_preimage(f, t))
} by {
    if subgroup_subset(subgroup_image(f, s), t) {
        forall(x: G) {
            if s.contains(x) {
                subgroup_image_contains_map(f, s, x)
                subgroup_subset(subgroup_image(f, s), t) = forall(y: H) {
                    subgroup_image(f, s).contains(y) implies t.contains(y)
                }
                subgroup_preimage_contains_eq(f, t, x)
                subgroup_preimage(f, t).contains(x)
            }
        }
        subgroup_subset(s, subgroup_preimage(f, t))
    }
    if subgroup_subset(s, subgroup_preimage(f, t)) {
        forall(y: H) {
            if subgroup_image(f, s).contains(y) {
                subgroup_image_contains_witness(f, s, y)
                let x: G satisfy { s.contains(x) and f.hom(x) = y }
                subgroup_subset(s, subgroup_preimage(f, t)) = forall(a: G) {
                    s.contains(a) implies subgroup_preimage(f, t).contains(a)
                }
                subgroup_preimage_contains_eq(f, t, x)
                t.contains(f.hom(x))
                t.contains(y)
            }
        }
        subgroup_subset(subgroup_image(f, s), t)
    }
}

/// Subgroup images are monotone with respect to source subgroup containment.
theorem subgroup_image_subset_of_subset[G: Group, H: Group](
    f: GroupHom[G, H],
    s: Subgroup[G],
    t: Subgroup[G]
) {
    subgroup_subset(s, t) implies subgroup_subset(subgroup_image(f, s), subgroup_image(f, t))
} by {
    if subgroup_subset(s, t) {
        forall(y: H) {
            if subgroup_image(f, s).contains(y) {
                subgroup_image_contains_witness(f, s, y)
                let x: G satisfy { s.contains(x) and f.hom(x) = y }
                subgroup_subset(s, t) = forall(a: G) {
                    s.contains(a) implies t.contains(a)
                }
                subgroup_image_contains_eq(f, t, y)
                subgroup_image(f, t).contains(y)
            }
        }
    }
}

from algebra.group.normal_subgroup import is_normal_subgroup, normal_conjugate_mem

/// Preimages of normal subgroups are normal.
theorem subgroup_preimage_normal[G: Group, H: Group](
    f: GroupHom[G, H],
    n: Subgroup[H]
) {
    is_normal_subgroup(n) implies is_normal_subgroup(subgroup_preimage(f, n))
} by {
    if is_normal_subgroup(n) {
        forall(g: G, a: G) {
            if subgroup_preimage(f, n).contains(a) {
                subgroup_preimage_contains_eq(f, n, a)
                group_hom_mul(f, g, a)
                group_hom_inv(f, g)
                group_hom_mul(f, g * a, g.inverse)
                normal_conjugate_mem(n, f.hom(g), f.hom(a))
                n.contains(f.hom(g) * f.hom(a) * f.hom(g).inverse)
                subgroup_preimage_contains_eq(f, n, g * a * g.inverse)
                subgroup_preimage(f, n).contains(g * a * g.inverse)
            }
        }
    }
}
