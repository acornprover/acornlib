/// Elementary order facts for group homomorphisms: homomorphisms send
/// identity powers to identity powers, injective homomorphisms preserve
/// element order exactly, and therefore isomorphisms (bijective
/// homomorphisms) preserve finite element order.

from algebra.group import Group, GroupHom, has_finite_order, group_hom_pow, group_hom_one
from data.basic.functions import is_injective_fn, is_bijection_fn, bijection_fn_is_injective
from nat import Nat

/// A homomorphism sends identity powers to identity powers.
theorem group_hom_pow_eq_one_of_pow_eq_one[G: Group, H: Group](f: GroupHom[G, H], a: G, n: Nat) {
    a.pow(n) = G.1 implies f.hom(a).pow(n) = H.1
} by {
    if a.pow(n) = G.1 {
        group_hom_pow(f, a, n)
        f.hom(a.pow(n)) = f.hom(a).pow(n)
        group_hom_one(f)
        f.hom(G.1) = H.1
        f.hom(a.pow(n)) = f.hom(G.1)
        f.hom(a).pow(n) = H.1
    }
}

/// An injective homomorphism reflects identity powers: the image has
/// `n`-th power the identity exactly when the source element does.
theorem injective_group_hom_pow_eq_one_iff[G: Group, H: Group](f: GroupHom[G, H], a: G, n: Nat) {
    is_injective_fn(f.hom) implies
    (f.hom(a).pow(n) = H.1) = (a.pow(n) = G.1)
} by {
    if is_injective_fn(f.hom) {
        if f.hom(a).pow(n) = H.1 {
            group_hom_pow(f, a, n)
            f.hom(a.pow(n)) = f.hom(a).pow(n)
            group_hom_one(f)
            f.hom(G.1) = H.1
            f.hom(a.pow(n)) = f.hom(G.1)
            is_injective_fn(f.hom) = forall(x: G, y: G) {
                f.hom(x) = f.hom(y) implies x = y
            }
            a.pow(n) = G.1
        }
        if a.pow(n) = G.1 {
            group_hom_pow_eq_one_of_pow_eq_one(f, a, n)
            f.hom(a).pow(n) = H.1
        }
        (f.hom(a).pow(n) = H.1) = (a.pow(n) = G.1)
    }
}

/// An injective group homomorphism preserves finite order: an element has
/// finite order exactly when its image does.
theorem injective_group_hom_preserves_finite_order[G: Group, H: Group](f: GroupHom[G, H], a: G) {
    is_injective_fn(f.hom) implies (has_finite_order(a) = has_finite_order(f.hom(a)))
} by {
    if is_injective_fn(f.hom) {
        has_finite_order(a) = exists(n: Nat) {
            n != Nat.0 and a.pow(n) = G.1
        }
        has_finite_order(f.hom(a)) = exists(n: Nat) {
            n != Nat.0 and f.hom(a).pow(n) = H.1
        }
        if has_finite_order(a) {
            exists(n: Nat) {
                n != Nat.0 and a.pow(n) = G.1
            }
            let n: Nat satisfy {
                n != Nat.0 and a.pow(n) = G.1
            }
            group_hom_pow_eq_one_of_pow_eq_one(f, a, n)
            f.hom(a).pow(n) = H.1
            exists(m: Nat) {
                m != Nat.0 and f.hom(a).pow(m) = H.1
            }
            has_finite_order(f.hom(a))
        }
        if has_finite_order(f.hom(a)) {
            exists(n: Nat) {
                n != Nat.0 and f.hom(a).pow(n) = H.1
            }
            let n: Nat satisfy {
                n != Nat.0 and f.hom(a).pow(n) = H.1
            }
            injective_group_hom_pow_eq_one_iff(f, a, n)
            (f.hom(a).pow(n) = H.1) = (a.pow(n) = G.1)
            a.pow(n) = G.1
            exists(m: Nat) {
                m != Nat.0 and a.pow(m) = G.1
            }
            has_finite_order(a)
        }
        has_finite_order(a) = has_finite_order(f.hom(a))
    }
}

/// An isomorphism (a bijective group homomorphism) preserves finite element
/// order.
theorem group_iso_preserves_finite_order[G: Group, H: Group](f: GroupHom[G, H], a: G) {
    is_bijection_fn(f.hom) implies (has_finite_order(a) = has_finite_order(f.hom(a)))
} by {
    if is_bijection_fn(f.hom) {
        bijection_fn_is_injective(f.hom)
        is_injective_fn(f.hom)
        injective_group_hom_preserves_finite_order(f, a)
        has_finite_order(a) = has_finite_order(f.hom(a))
    }
}
