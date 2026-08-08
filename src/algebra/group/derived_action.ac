from algebra.group import Group, GroupHom, group_hom_mul, group_hom_one
from algebra.group_action import MulAction, is_mul_action, action_identity_constraint,
    action_mul_constraint, mul_action_one, mul_action_mul, is_equivariant_map,
    is_transitive_action, is_faithful_action, faithful_action_eq_of_pointwise_eq,
    orbit, orbit_contains_self, orbit_contains_witness, orbit_contains_action,
    stabilizer, stabilizer_fixes, stabilizer_contains_of_fixes
from data.basic.functions import is_injective_fn, is_surjective_fn, injective_fn_eq, surjective_fn_has_preimage

/// The trivial action map fixes every point.
define trivial_act[G: Group, X](g: G, x: X) -> X {
    x
}

/// The trivial action map satisfies the identity constraint.
theorem trivial_act_identity_constraint[G: Group, X] {
    action_identity_constraint(trivial_act[G, X])
} by {
    forall(x: X) {
        trivial_act[G, X](G.1, x) = x
    }
}

/// The trivial action map satisfies the multiplication constraint.
theorem trivial_act_mul_constraint[G: Group, X] {
    action_mul_constraint(trivial_act[G, X])
} by {
    forall(g: G, h: G, x: X) {
        trivial_act[G, X](g, trivial_act[G, X](h, x)) = x
    }
}

/// The trivial action map is a left action.
theorem trivial_act_is_mul_action[G: Group, X] {
    is_mul_action(trivial_act[G, X])
} by {
    trivial_act_identity_constraint[G, X]
    trivial_act_mul_constraint[G, X]
}

/// The bundled trivial action of G on X.
let trivial_action[G: Group, X]: MulAction[G, X] satisfy {
    MulAction.new(trivial_act[G, X]) = Option.some(trivial_action)
}

/// The action map of `trivial_action` is `trivial_act`.
theorem trivial_action_act[G: Group, X](g: G, x: X) {
    trivial_action[G, X].act(g, x) = x
}

/// In the trivial action every orbit is a singleton.
theorem trivial_action_orbit_contains_iff[G: Group, X](x: X, y: X) {
    orbit(trivial_action[G, X], x).contains(y) iff x = y
} by {
    if orbit(trivial_action[G, X], x).contains(y) {
        orbit_contains_witness(trivial_action[G, X], x, y)
        let g: G satisfy {
            y = trivial_action[G, X].act(g, x)
        }
        trivial_action_act(g, x)
        x = y
    }
    if x = y {
        orbit_contains_self(trivial_action[G, X], x)
        orbit(trivial_action[G, X], x).contains(y)
    }
}

/// The trivial action on a singleton-style situation: it is transitive when any two points coincide.
theorem trivial_action_is_transitive_of_subsingleton[G: Group, X] {
    (forall(x: X, y: X) { x = y }) implies is_transitive_action(trivial_action[G, X])
} by {
    if forall(x: X, y: X) { x = y } {
        forall(x: X, y: X) {
            trivial_action_orbit_contains_iff[G, X](x, y)
            orbit(trivial_action[G, X], x).contains(y)
        }
        is_transitive_action(trivial_action[G, X]) = forall(x: X, y: X) {
            orbit(trivial_action[G, X], x).contains(y)
        }
    }
}

/// Every group element belongs to the stabilizer of every point in the trivial action.
theorem trivial_action_stabilizer_contains[G: Group, X](x: X, g: G) {
    stabilizer(trivial_action[G, X], x).contains(g)
} by {
    trivial_action_act(g, x)
    stabilizer_contains_of_fixes(trivial_action[G, X], x, g)
}

/// The restricted action map: pull a G-action back along a homomorphism `phi: H -> G`.
define restricted_act[H: Group, G: Group, X](phi: GroupHom[H, G], a: MulAction[G, X],
    h: H, x: X) -> X {
    a.act(phi.hom(h), x)
}

/// The restricted action map satisfies the identity constraint.
theorem restricted_act_identity_constraint[H: Group, G: Group, X](phi: GroupHom[H, G],
    a: MulAction[G, X]) {
    action_identity_constraint(restricted_act(phi, a))
} by {
    forall(x: X) {
        group_hom_one(phi)
        mul_action_one(a, x)
        restricted_act(phi, a, H.1, x) = x
    }
}

/// The restricted action map applied at h*k factors through phi.hom on each factor.
theorem restricted_act_mul_apply[H: Group, G: Group, X](phi: GroupHom[H, G],
    a: MulAction[G, X], h1: H, h2: H, x: X) {
    restricted_act(phi, a, h1 * h2, x) = a.act(phi.hom(h1), a.act(phi.hom(h2), x))
} by {
    group_hom_mul(phi, h1, h2)
    mul_action_mul(a, phi.hom(h1), phi.hom(h2), x)
}

/// The restricted action map composed twice equals the action of h1 on the action of h2.
theorem restricted_act_compose[H: Group, G: Group, X](phi: GroupHom[H, G],
    a: MulAction[G, X], h1: H, h2: H, x: X) {
    restricted_act(phi, a, h1, restricted_act(phi, a, h2, x)) = a.act(phi.hom(h1),
        a.act(phi.hom(h2), x))
} by {
    let z: X = restricted_act(phi, a, h2, x)
}

/// The restricted action map satisfies the multiplication constraint.
theorem restricted_act_mul_constraint[H: Group, G: Group, X](phi: GroupHom[H, G],
    a: MulAction[G, X]) {
    action_mul_constraint(restricted_act(phi, a))
} by {
    forall(h1: H, h2: H, x: X) {
        restricted_act_mul_apply(phi, a, h1, h2, x)
        restricted_act_compose(phi, a, h1, h2, x)
    }
}

/// The restricted action map is a left action.
theorem restricted_act_is_mul_action[H: Group, G: Group, X](phi: GroupHom[H, G],
    a: MulAction[G, X]) {
    is_mul_action(restricted_act(phi, a))
} by {
    restricted_act_identity_constraint(phi, a)
    restricted_act_mul_constraint(phi, a)
}

/// The bundled restricted action obtained by pulling back along a group homomorphism.
let restricted_action[H: Group, G: Group, X](phi: GroupHom[H, G], a: MulAction[G, X]) -> result: MulAction[H, X] satisfy {
    MulAction.new(restricted_act(phi, a)) = Option.some(result)
} by {
    restricted_act_is_mul_action(phi, a)
}

/// The action map of `restricted_action` evaluates through `phi.hom`.
theorem restricted_action_act[H: Group, G: Group, X](phi: GroupHom[H, G],
    a: MulAction[G, X], h: H, x: X) {
    restricted_action(phi, a).act(h, x) = a.act(phi.hom(h), x)
} by {
    restricted_action(phi, a).act(h, x) = restricted_act(phi, a, h, x)
}

/// Every orbit in the restricted action is contained in the corresponding orbit of the original action.
theorem restricted_action_orbit_subset[H: Group, G: Group, X](phi: GroupHom[H, G],
    a: MulAction[G, X], x: X, y: X) {
    orbit(restricted_action(phi, a), x).contains(y) implies orbit(a, x).contains(y)
} by {
    if orbit(restricted_action(phi, a), x).contains(y) {
        orbit_contains_witness(restricted_action(phi, a), x, y)
        let h: H satisfy {
            y = restricted_action(phi, a).act(h, x)
        }
        restricted_action_act(phi, a, h, x)
        orbit_contains_action(a, x, phi.hom(h))
        orbit(a, x).contains(y)
    }
}

/// Pointwise equality of the restricted action follows from equality of `phi.hom`.
theorem restricted_action_pointwise_eq_of_phi_eq[H: Group, G: Group, X](
    phi: GroupHom[H, G], a: MulAction[G, X], h1: H, h2: H, x: X) {
    phi.hom(h1) = phi.hom(h2) implies
        restricted_action(phi, a).act(h1, x) = restricted_action(phi, a).act(h2, x)
} by {
    if phi.hom(h1) = phi.hom(h2) {
        restricted_action_act(phi, a, h1, x)
        restricted_action_act(phi, a, h2, x)
    }
}

/// Membership in the stabilizer of the restricted action lifts to membership in the stabilizer
/// of the original action via `phi.hom`.
theorem restricted_action_stabilizer_to_original[H: Group, G: Group, X](
    phi: GroupHom[H, G], a: MulAction[G, X], x: X, h: H) {
    stabilizer(restricted_action(phi, a), x).contains(h) implies
        stabilizer(a, x).contains(phi.hom(h))
} by {
    if stabilizer(restricted_action(phi, a), x).contains(h) {
        stabilizer_fixes(restricted_action(phi, a), x, h)
        restricted_action_act(phi, a, h, x)
        stabilizer_contains_of_fixes(a, x, phi.hom(h))
        stabilizer(a, x).contains(phi.hom(h))
    }
}

/// Membership in the stabilizer of the original action at `phi.hom(h)` reflects to membership
/// in the stabilizer of the restricted action at `h`.
theorem restricted_action_stabilizer_from_original[H: Group, G: Group, X](
    phi: GroupHom[H, G], a: MulAction[G, X], x: X, h: H) {
    stabilizer(a, x).contains(phi.hom(h)) implies
        stabilizer(restricted_action(phi, a), x).contains(h)
} by {
    if stabilizer(a, x).contains(phi.hom(h)) {
        stabilizer_fixes(a, x, phi.hom(h))
        restricted_action_act(phi, a, h, x)
        stabilizer_contains_of_fixes(restricted_action(phi, a), x, h)
        stabilizer(restricted_action(phi, a), x).contains(h)
    }
}

/// Stabilizer membership for the restricted action is exactly preimage of the original stabilizer
/// under `phi.hom`.
theorem restricted_action_stabilizer_iff[H: Group, G: Group, X](
    phi: GroupHom[H, G], a: MulAction[G, X], x: X, h: H) {
    stabilizer(restricted_action(phi, a), x).contains(h) iff
        stabilizer(a, x).contains(phi.hom(h))
} by {
    if stabilizer(restricted_action(phi, a), x).contains(h) {
        restricted_action_stabilizer_to_original(phi, a, x, h)
    }
    if stabilizer(a, x).contains(phi.hom(h)) {
        restricted_action_stabilizer_from_original(phi, a, x, h)
    }
}

/// If the restricted action is faithful then `phi` is injective.
theorem restricted_action_faithful_implies_phi_injective[H: Group, G: Group, X](
    phi: GroupHom[H, G], a: MulAction[G, X], h1: H, h2: H) {
    is_faithful_action(restricted_action(phi, a)) and phi.hom(h1) = phi.hom(h2) implies h1 = h2
} by {
    if is_faithful_action(restricted_action(phi, a)) and phi.hom(h1) = phi.hom(h2) {
        forall(x: X) {
            restricted_action_pointwise_eq_of_phi_eq(phi, a, h1, h2, x)
        }
        faithful_action_eq_of_pointwise_eq(restricted_action(phi, a), h1, h2)
    }
}

/// A transitive action stays transitive after restriction along a surjective group homomorphism.
theorem restricted_action_is_transitive_of_surjective[H: Group, G: Group, X](
    phi: GroupHom[H, G], a: MulAction[G, X]) {
    is_surjective_fn(phi.hom) and is_transitive_action(a) implies
        is_transitive_action(restricted_action(phi, a))
} by {
    if is_surjective_fn(phi.hom) and is_transitive_action(a) {
        forall(x: X, y: X) {
            orbit(a, x).contains(y)
            orbit_contains_witness(a, x, y)
            let g: G satisfy {
                y = a.act(g, x)
            }
            surjective_fn_has_preimage(phi.hom, g)
            let h: H satisfy {
                phi.hom(h) = g
            }
            restricted_action_act(phi, a, h, x)
            orbit_contains_action(restricted_action(phi, a), x, h)
            orbit(restricted_action(phi, a), x).contains(y)
        }
        is_transitive_action(restricted_action(phi, a))
    }
}

/// A faithful action restricts to a faithful action along an injective group homomorphism.
theorem restricted_action_is_faithful_of_injective[H: Group, G: Group, X](
    phi: GroupHom[H, G], a: MulAction[G, X]) {
    is_injective_fn(phi.hom) and is_faithful_action(a) implies
        is_faithful_action(restricted_action(phi, a))
} by {
    if is_injective_fn(phi.hom) and is_faithful_action(a) {
        forall(h1: H, h2: H) {
            if forall(x: X) { restricted_action(phi, a).act(h1, x) =
                    restricted_action(phi, a).act(h2, x) } {
                forall(x: X) {
                    restricted_action_act(phi, a, h1, x)
                    restricted_action_act(phi, a, h2, x)
                    a.act(phi.hom(h1), x) = a.act(phi.hom(h2), x)
                }
                faithful_action_eq_of_pointwise_eq(a, phi.hom(h1), phi.hom(h2))
                injective_fn_eq(phi.hom, h1, h2)
                h1 = h2
            }
        }
        is_faithful_action(restricted_action(phi, a))
    }
}
