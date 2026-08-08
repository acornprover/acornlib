/// Fixed-point coordinates for product group actions.

from algebra.group import Group
from pair import Pair, pair_ext
from algebra.group_action import MulAction, fixed_points, fixed_points_contains_eq
from algebra.group.product_action import product_action, product_action_first, product_action_second

/// A product-action fixed point has a fixed first coordinate.
theorem product_action_fixed_points_first[G: Group, X, Y](
    a: MulAction[G, X],
    b: MulAction[G, Y],
    g: G,
    p: Pair[X, Y]
) {
    fixed_points(product_action(a, b), g).contains(p) implies
        fixed_points(a, g).contains(p.first)
} by {
    if fixed_points(product_action(a, b), g).contains(p) {
        fixed_points_contains_eq(product_action(a, b), g, p)
        product_action(a, b).act(g, p) = p
        product_action_first(a, b, g, p)
        product_action(a, b).act(g, p).first = a.act(g, p.first)
        product_action(a, b).act(g, p).first = p.first
        a.act(g, p.first) = p.first
        fixed_points_contains_eq(a, g, p.first)
        fixed_points(a, g).contains(p.first)
    }
}

/// A product-action fixed point has a fixed second coordinate.
theorem product_action_fixed_points_second[G: Group, X, Y](
    a: MulAction[G, X],
    b: MulAction[G, Y],
    g: G,
    p: Pair[X, Y]
) {
    fixed_points(product_action(a, b), g).contains(p) implies
        fixed_points(b, g).contains(p.second)
} by {
    if fixed_points(product_action(a, b), g).contains(p) {
        fixed_points_contains_eq(product_action(a, b), g, p)
        product_action(a, b).act(g, p) = p
        product_action_second(a, b, g, p)
        product_action(a, b).act(g, p).second = b.act(g, p.second)
        product_action(a, b).act(g, p).second = p.second
        b.act(g, p.second) = p.second
        fixed_points_contains_eq(b, g, p.second)
        fixed_points(b, g).contains(p.second)
    }
}

/// Fixed coordinates make the pair fixed by the product action.
theorem product_action_fixed_points_of_coordinates[G: Group, X, Y](
    a: MulAction[G, X],
    b: MulAction[G, Y],
    g: G,
    p: Pair[X, Y]
) {
    fixed_points(a, g).contains(p.first) and fixed_points(b, g).contains(p.second) implies
        fixed_points(product_action(a, b), g).contains(p)
} by {
    if fixed_points(a, g).contains(p.first) and fixed_points(b, g).contains(p.second) {
        fixed_points_contains_eq(a, g, p.first)
        a.act(g, p.first) = p.first
        fixed_points_contains_eq(b, g, p.second)
        b.act(g, p.second) = p.second
        let q = product_action(a, b).act(g, p)
        product_action_first(a, b, g, p)
        q.first = a.act(g, p.first)
        q.first = p.first
        product_action_second(a, b, g, p)
        q.second = b.act(g, p.second)
        q.second = p.second
        pair_ext(q, p)
        q = p
        fixed_points_contains_eq(product_action(a, b), g, p)
        fixed_points(product_action(a, b), g).contains(p)
    }
}

/// Fixed points of a product action are exactly pairs of fixed coordinates.
theorem product_action_fixed_points_contains_eq_coordinates[G: Group, X, Y](
    a: MulAction[G, X],
    b: MulAction[G, Y],
    g: G,
    p: Pair[X, Y]
) {
    fixed_points(product_action(a, b), g).contains(p) =
        (fixed_points(a, g).contains(p.first) and fixed_points(b, g).contains(p.second))
} by {
    if fixed_points(product_action(a, b), g).contains(p) {
        product_action_fixed_points_first(a, b, g, p)
        product_action_fixed_points_second(a, b, g, p)
        fixed_points(a, g).contains(p.first) and fixed_points(b, g).contains(p.second)
    }
    if fixed_points(a, g).contains(p.first) and fixed_points(b, g).contains(p.second) {
        product_action_fixed_points_of_coordinates(a, b, g, p)
        fixed_points(product_action(a, b), g).contains(p)
    }
    fixed_points(product_action(a, b), g).contains(p) =
        (fixed_points(a, g).contains(p.first) and fixed_points(b, g).contains(p.second))
}
