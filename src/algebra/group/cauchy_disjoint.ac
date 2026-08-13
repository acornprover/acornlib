/// Pairwise disjointness of the prepended sublists used in the tuple
/// enumeration, isolated in a small module so that proof search stays fast.

from list import List, map, map_contains
from nat import Nat

numerals Nat

/// All lists of length `n` over `items`.
define all_lists[T](items: List[T], n: Nat) -> List[List[T]] {
    match n {
        Nat.zero {
            List.singleton[List[T]](List.nil[T])
        }
        Nat.suc(m) {
            List.nil[List[T]]
        }
    }
}

/// All lists obtained by prepending `x` to each list in `ts`.
define prepend_all[T](x: T, ts: List[List[T]]) -> List[List[T]] {
    map[List[T], List[T]](ts, function(t: List[T]) {
        List.cons(x, t)
    })
}

/// Prepending unfolds to the mapped cons list.
theorem prepend_all_unfold[T](x: T, ts: List[List[T]]) {
    prepend_all(x, ts) = map[List[T], List[T]](ts, function(t: List[T]) {
        List.cons(x, t)
    })
} by {
}

/// Membership in the prepended lists exhibits the prepended head and tail.
theorem prepend_all_contains_witness[T](x: T, u: List[T], ll: List[List[T]]) {
    prepend_all(x, ll).contains(u) implies exists(t1: List[T]) { ll.contains(t1) and List.cons(x, t1) = u }
} by {
    if prepend_all(x, ll).contains(u) {
        prepend_all_unfold(x, ll)
        prepend_all(x, ll) = map[List[T], List[T]](ll, function(t: List[T]) {
            List.cons(x, t)
        })
        map_contains(ll, function(t: List[T]) { List.cons(x, t) }, u)
        exists(t1: List[T]) { ll.contains(t1) and List.cons(x, t1) = u }
    }
}

/// Distinct prepended heads give disjoint prepended sublists.
theorem prepend_all_pairwise_disjoint[T](items: List[T], k: Nat) {
    forall(x: T, y: T, u: List[T]) {
        x != y implies
            not (prepend_all(x, all_lists(items, k)).contains(u) and
                prepend_all(y, all_lists(items, k)).contains(u))
    }
} by {
    forall(x: T, y: T, u: List[T]) {
        if x != y {
            if prepend_all(x, all_lists(items, k)).contains(u) and
                prepend_all(y, all_lists(items, k)).contains(u) {
                prepend_all_contains_witness(x, u, all_lists(items, k))
                let t1: List[T] satisfy {
                    all_lists(items, k).contains(t1) and List.cons(x, t1) = u
                }
                List.cons(x, t1) = u
                prepend_all_contains_witness(y, u, all_lists(items, k))
                let t2: List[T] satisfy {
                    all_lists(items, k).contains(t2) and List.cons(y, t2) = u
                }
                List.cons(y, t2) = u
                List.cons(x, t1) = List.cons(y, t2)
                x = y
                x != y
                false
            }
            not (prepend_all(x, all_lists(items, k)).contains(u) and
                prepend_all(y, all_lists(items, k)).contains(u))
        }
    }
}
