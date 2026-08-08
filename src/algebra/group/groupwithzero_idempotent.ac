/// Idempotent elements in multiplicative monoids with zero.

from algebra.monoid.monoid import Monoid
from algebra.zero import Zero
from algebra.idempotent_elem import is_idempotent_elem, one_is_idempotent_elem,
    idempotent_elem_mul_self, is_idempotent_elem_of_mul_self

/// A multiplicative monoid with an absorbing zero element.
typeclass M: MonoidWithZero extends Monoid, Zero {
    /// Zero is absorbing on the left for multiplication.
    zero_mul(a: M) {
        M.0 * a = M.0
    }

    /// Zero is absorbing on the right for multiplication.
    mul_zero(a: M) {
        a * M.0 = M.0
    }
}

/// A monoid with zero where multiplication by a nonzero left factor is cancellative.
typeclass M: LeftCancelMonoidWithZero extends MonoidWithZero {
    /// A nonzero left factor may be cancelled from a multiplication equality.
    mul_left_cancel_of_ne_zero(a: M, b: M, c: M) {
        a != M.0 and a * b = a * c implies b = c
    }
}

/// Zero is an idempotent element in a monoid with zero.
theorem zero_is_idempotent_elem[M: MonoidWithZero] {
    is_idempotent_elem(M.0)
} by {
    M.zero_mul(M.0)
    is_idempotent_elem_of_mul_self[M](M.0)
}

/// In a left-cancellative monoid with zero, idempotents are exactly zero or one.
theorem idempotent_elem_iff_eq_zero_or_one[M: LeftCancelMonoidWithZero](p: M) {
    is_idempotent_elem(p) iff p = M.0 or p = M.1
} by {
    if is_idempotent_elem(p) {
        if p = M.0 {
        } else {
            p != M.0
            idempotent_elem_mul_self[M](p)
            p * p = p
            p * M.1 = p
            p * p = p * M.1
            M.mul_left_cancel_of_ne_zero(p, p, M.1)
            p = M.1
        }
    }
    if p = M.0 or p = M.1 {
        if p = M.0 {
            zero_is_idempotent_elem[M]
            is_idempotent_elem(M.0)
            is_idempotent_elem(p)
        } else {
            p = M.1
            one_is_idempotent_elem[M]
            is_idempotent_elem(M.1)
            is_idempotent_elem(p)
        }
    }
}
