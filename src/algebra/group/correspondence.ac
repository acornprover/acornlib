/// The correspondence theorem (lattice isomorphism): for a normal subgroup
/// `N` of `G`, the map sending a subgroup `S` of `G` containing `N` to its
/// image under the projection `G -> G/N` is a bijection between the subgroups
/// of `G` containing `N` and the subgroups of the quotient `G/N`.
///
/// Subgroups of the quotient are represented by raw predicates on
/// `QuotientOver[G]` closed under the quotient operations (the quotient is not
/// a bundled `Group` instance), and the preimage of such a predicate is a
/// bundled `Subgroup[G]` containing `N`.  The two constructions are inverse to
/// each other: `π⁻¹(π(S)) = S` for `N ≤ S`, and `π(π⁻¹(T)) = T` for every
/// subgroup `T` of `G/N`.

from algebra.group import Group, inverse_mul, inverse_inverse
from algebra.subgroup import Subgroup, subgroup_subset, subgroup_mul_mem,
    subgroup_inv_mem, subgroup_contains_identity, subgroup_constraint,
    identity_constraint, closure_constraint, inverse_constraint
from algebra.group.normal_subgroup import is_normal_subgroup,
    normal_subgroup_quotient_project, normal_subgroup_quotient_project_eq_mk,
    normal_subgroup_quotient_relation, normal_subgroup_quotient_relation_rel,
    normal_subgroup_quotient_one, normal_subgroup_quotient_project_one,
    normal_subgroup_quotient_mul, normal_subgroup_quotient_project_mul,
    normal_subgroup_quotient_inverse, normal_subgroup_quotient_project_inverse,
    normal_subgroup_quotient_project_eq_one_iff_contains,
    normal_subgroup_rel_of_quotient_project_eq, normal_subgroup_rel,
    normal_subgroup_contains_inverse_mul_of_rel
from algebra.group.third_iso import quotient_subgroup_constraint,
    quotient_subgroup_mul_closed, quotient_subgroup_inv_closed
from data.basic.equivalence import QuotientOver, quotient_over_mk_surjective,
    quotient_over_mk_fields, quotient_over_mk

// ==== Image of a subgroup under the projection ====

/// Membership of a quotient element in the image of a subgroup `S` under the
/// projection to `G/N`: the quotient element is the projection of an element
/// of `S`.
define projection_image_contains[G: Group](n: Subgroup[G], s: Subgroup[G], q: QuotientOver[G]) -> Bool {
    exists(a: G) {
        s.contains(a) and normal_subgroup_quotient_project(n, a) = q
    }
}

/// The image of a subgroup under the projection contains the quotient identity.
theorem projection_image_identity[G: Group](n: Subgroup[G], s: Subgroup[G]) {
    projection_image_contains(n, s, normal_subgroup_quotient_one(n))
} by {
    subgroup_contains_identity(s)
    normal_subgroup_quotient_project_one(n)
    normal_subgroup_quotient_project(n, G.1) = normal_subgroup_quotient_one(n)
    projection_image_contains(n, s, normal_subgroup_quotient_one(n))
}

/// The image of a subgroup under the projection is closed under quotient
/// multiplication.
theorem projection_image_mul[G: Group](n: Subgroup[G], s: Subgroup[G], q1: QuotientOver[G], q2: QuotientOver[G]) {
    is_normal_subgroup(n) and projection_image_contains(n, s, q1) and projection_image_contains(n, s, q2) implies
    projection_image_contains(n, s, normal_subgroup_quotient_mul(n, q1, q2))
} by {
    if is_normal_subgroup(n) and projection_image_contains(n, s, q1) and projection_image_contains(n, s, q2) {
        projection_image_contains(n, s, q1)
        projection_image_contains(n, s, q2)
        projection_image_contains(n, s, q1) = exists(a: G) {
            s.contains(a) and normal_subgroup_quotient_project(n, a) = q1
        }
        projection_image_contains(n, s, q2) = exists(a: G) {
            s.contains(a) and normal_subgroup_quotient_project(n, a) = q2
        }
        let a: G satisfy {
            s.contains(a) and normal_subgroup_quotient_project(n, a) = q1
        }
        let b: G satisfy {
            s.contains(b) and normal_subgroup_quotient_project(n, b) = q2
        }
        subgroup_mul_mem(s, a, b)
        normal_subgroup_quotient_project_mul(n, a, b)
        normal_subgroup_quotient_project(n, a * b) =
            normal_subgroup_quotient_mul(n,
                normal_subgroup_quotient_project(n, a),
                normal_subgroup_quotient_project(n, b))
        normal_subgroup_quotient_mul(n, q1, q2) =
            normal_subgroup_quotient_mul(n,
                normal_subgroup_quotient_project(n, a),
                normal_subgroup_quotient_project(n, b))
        normal_subgroup_quotient_project(n, a * b) =
            normal_subgroup_quotient_mul(n, q1, q2)
        projection_image_contains(n, s, normal_subgroup_quotient_mul(n, q1, q2))
    }
}

/// The image of a subgroup under the projection is closed under quotient
/// inversion.
theorem projection_image_inverse[G: Group](n: Subgroup[G], s: Subgroup[G], q: QuotientOver[G]) {
    is_normal_subgroup(n) and projection_image_contains(n, s, q) implies
    projection_image_contains(n, s, normal_subgroup_quotient_inverse(n, q))
} by {
    if is_normal_subgroup(n) and projection_image_contains(n, s, q) {
        projection_image_contains(n, s, q) = exists(a: G) {
            s.contains(a) and normal_subgroup_quotient_project(n, a) = q
        }
        let a: G satisfy {
            s.contains(a) and normal_subgroup_quotient_project(n, a) = q
        }
        subgroup_inv_mem(s, a)
        normal_subgroup_quotient_project_inverse(n, a)
        normal_subgroup_quotient_project(n, a.inverse) =
            normal_subgroup_quotient_inverse(n, normal_subgroup_quotient_project(n, a))
        normal_subgroup_quotient_inverse(n, q) =
            normal_subgroup_quotient_inverse(n, normal_subgroup_quotient_project(n, a))
        normal_subgroup_quotient_project(n, a.inverse) =
            normal_subgroup_quotient_inverse(n, q)
        projection_image_contains(n, s, normal_subgroup_quotient_inverse(n, q))
    }
}

/// The image of a subgroup of `G` under the projection is a subgroup of
/// `G/N`.
theorem projection_image_constraint[G: Group](n: Subgroup[G], s: Subgroup[G]) {
    is_normal_subgroup(n) implies quotient_subgroup_constraint(n, projection_image_contains(n, s))
} by {
    if is_normal_subgroup(n) {
        projection_image_identity(n, s)
        projection_image_contains(n, s, normal_subgroup_quotient_one(n))
        forall(q1: QuotientOver[G], q2: QuotientOver[G]) {
            if projection_image_contains(n, s, q1) and projection_image_contains(n, s, q2) {
                projection_image_mul(n, s, q1, q2)
                projection_image_contains(n, s, normal_subgroup_quotient_mul(n, q1, q2))
            }
        }
        quotient_subgroup_mul_closed(n, projection_image_contains(n, s))
        forall(q: QuotientOver[G]) {
            if projection_image_contains(n, s, q) {
                projection_image_inverse(n, s, q)
                projection_image_contains(n, s, normal_subgroup_quotient_inverse(n, q))
            }
        }
        quotient_subgroup_inv_closed(n, projection_image_contains(n, s))
        quotient_subgroup_constraint(n, projection_image_contains(n, s))
    }
}

// ==== Preimage of a quotient subgroup under the projection ====

/// Membership in the preimage of a predicate on the quotient `G/N` under the
/// projection: the projected element belongs to the predicate.
define projection_preimage_contains[G: Group](n: Subgroup[G], t: QuotientOver[G] -> Bool, g: G) -> Bool {
    t(normal_subgroup_quotient_project(n, g))
}

/// The preimage of a subgroup of `G/N` contains the identity.
theorem projection_preimage_identity_constraint[G: Group](n: Subgroup[G], t: QuotientOver[G] -> Bool) {
    t(normal_subgroup_quotient_one(n)) implies
    identity_constraint(projection_preimage_contains(n, t))
} by {
    if t(normal_subgroup_quotient_one(n)) {
        normal_subgroup_quotient_project_one(n)
        normal_subgroup_quotient_project(n, G.1) = normal_subgroup_quotient_one(n)
        t(normal_subgroup_quotient_project(n, G.1))
        projection_preimage_contains(n, t, G.1)
        identity_constraint(projection_preimage_contains(n, t))
    }
}

/// The preimage of a subgroup of `G/N` is closed under multiplication.
theorem projection_preimage_closure_constraint[G: Group](n: Subgroup[G], t: QuotientOver[G] -> Bool) {
    is_normal_subgroup(n) and
    (forall(a: QuotientOver[G], b: QuotientOver[G]) {
        t(a) and t(b) implies t(normal_subgroup_quotient_mul(n, a, b))
    }) implies
    closure_constraint(projection_preimage_contains(n, t))
} by {
    if is_normal_subgroup(n) and
        (forall(a: QuotientOver[G], b: QuotientOver[G]) {
            t(a) and t(b) implies t(normal_subgroup_quotient_mul(n, a, b))
        }) {
        forall(g: G, h: G) {
            if projection_preimage_contains(n, t, g) and projection_preimage_contains(n, t, h) {
                t(normal_subgroup_quotient_project(n, g))
                t(normal_subgroup_quotient_project(n, h))
                normal_subgroup_quotient_project_mul(n, g, h)
                normal_subgroup_quotient_project(n, g * h) =
                    normal_subgroup_quotient_mul(n,
                        normal_subgroup_quotient_project(n, g),
                        normal_subgroup_quotient_project(n, h))
                t(normal_subgroup_quotient_mul(n,
                    normal_subgroup_quotient_project(n, g),
                    normal_subgroup_quotient_project(n, h)))
                t(normal_subgroup_quotient_project(n, g * h))
                projection_preimage_contains(n, t, g * h)
            }
        }
        closure_constraint(projection_preimage_contains(n, t))
    }
}

/// The preimage of a subgroup of `G/N` is closed under inverses.
theorem projection_preimage_inverse_constraint[G: Group](n: Subgroup[G], t: QuotientOver[G] -> Bool) {
    is_normal_subgroup(n) and
    (forall(a: QuotientOver[G]) {
        t(a) implies t(normal_subgroup_quotient_inverse(n, a))
    }) implies
    inverse_constraint(projection_preimage_contains(n, t))
} by {
    if is_normal_subgroup(n) and
        (forall(a: QuotientOver[G]) {
            t(a) implies t(normal_subgroup_quotient_inverse(n, a))
        }) {
        forall(g: G) {
            if projection_preimage_contains(n, t, g) {
                t(normal_subgroup_quotient_project(n, g))
                normal_subgroup_quotient_project_inverse(n, g)
                normal_subgroup_quotient_project(n, g.inverse) =
                    normal_subgroup_quotient_inverse(n,
                        normal_subgroup_quotient_project(n, g))
                t(normal_subgroup_quotient_inverse(n,
                    normal_subgroup_quotient_project(n, g)))
                t(normal_subgroup_quotient_project(n, g.inverse))
                projection_preimage_contains(n, t, g.inverse)
            }
        }
        inverse_constraint(projection_preimage_contains(n, t))
    }
}

/// The preimage of a subgroup of `G/N` under the projection is a subgroup
/// of `G`.
theorem projection_preimage_constraint[G: Group](n: Subgroup[G], t: QuotientOver[G] -> Bool) {
    is_normal_subgroup(n) and t(normal_subgroup_quotient_one(n)) and
    (forall(a: QuotientOver[G], b: QuotientOver[G]) {
        t(a) and t(b) implies t(normal_subgroup_quotient_mul(n, a, b))
    }) and
    (forall(a: QuotientOver[G]) {
        t(a) implies t(normal_subgroup_quotient_inverse(n, a))
    }) implies
    subgroup_constraint(projection_preimage_contains(n, t))
} by {
    if is_normal_subgroup(n) and t(normal_subgroup_quotient_one(n)) and
        (forall(a: QuotientOver[G], b: QuotientOver[G]) {
            t(a) and t(b) implies t(normal_subgroup_quotient_mul(n, a, b))
        }) and
        (forall(a: QuotientOver[G]) {
            t(a) implies t(normal_subgroup_quotient_inverse(n, a))
        }) {
        projection_preimage_identity_constraint(n, t)
        identity_constraint(projection_preimage_contains(n, t))
        projection_preimage_closure_constraint(n, t)
        closure_constraint(projection_preimage_contains(n, t))
        projection_preimage_inverse_constraint(n, t)
        inverse_constraint(projection_preimage_contains(n, t))
        subgroup_constraint(projection_preimage_contains(n, t))
    }
}

/// The preimage of a subgroup of `G/N` contains `N`.
theorem projection_preimage_contains_n[G: Group](n: Subgroup[G], t: QuotientOver[G] -> Bool) {
    t(normal_subgroup_quotient_one(n)) implies
    forall(g: G) {
        n.contains(g) implies projection_preimage_contains(n, t, g)
    }
} by {
    if t(normal_subgroup_quotient_one(n)) {
        forall(g: G) {
            if n.contains(g) {
                normal_subgroup_quotient_project_eq_one_iff_contains(n, g)
                normal_subgroup_quotient_project(n, g) = normal_subgroup_quotient_one(n)
                t(normal_subgroup_quotient_project(n, g))
                projection_preimage_contains(n, t, g)
            }
        }
    }
}

// ==== The correspondence ====

/// The image-preimage adjunction: `π(S) ⊆ T` exactly when `S ⊆ π⁻¹(T)`.
theorem projection_image_subset_iff_preimage_subset[G: Group](n: Subgroup[G], s: Subgroup[G], t: QuotientOver[G] -> Bool) {
    (forall(q: QuotientOver[G]) {
        projection_image_contains(n, s, q) implies t(q)
    }) = (forall(g: G) {
        s.contains(g) implies projection_preimage_contains(n, t, g)
    })
} by {
    if forall(q: QuotientOver[G]) {
        projection_image_contains(n, s, q) implies t(q)
    } {
        forall(g: G) {
            if s.contains(g) {
                projection_image_contains(n, s, normal_subgroup_quotient_project(n, g))
                t(normal_subgroup_quotient_project(n, g))
                projection_preimage_contains(n, t, g)
            }
        }
    }
    if forall(g: G) {
        s.contains(g) implies projection_preimage_contains(n, t, g)
    } {
        forall(q: QuotientOver[G]) {
            if projection_image_contains(n, s, q) {
                projection_image_contains(n, s, q) = exists(a: G) {
                    s.contains(a) and normal_subgroup_quotient_project(n, a) = q
                }
                let a: G satisfy {
                    s.contains(a) and normal_subgroup_quotient_project(n, a) = q
                }
                projection_preimage_contains(n, t, a)
                t(normal_subgroup_quotient_project(n, a))
                t(q)
            }
        }
    }
}

/// The left round trip: for a subgroup `S` containing `N`, the preimage of
/// the image of `S` is `S` itself.
theorem projection_left_inverse[G: Group](n: Subgroup[G], s: Subgroup[G], g: G) {
    subgroup_subset(n, s) implies
    projection_preimage_contains(n, projection_image_contains(n, s), g) = s.contains(g)
} by {
    if subgroup_subset(n, s) {
        if projection_preimage_contains(n, projection_image_contains(n, s), g) {
            projection_image_contains(n, s, normal_subgroup_quotient_project(n, g)) =
                exists(a: G) {
                    s.contains(a) and normal_subgroup_quotient_project(n, a) =
                        normal_subgroup_quotient_project(n, g)
                }
            projection_image_contains(n, s, normal_subgroup_quotient_project(n, g))
            exists(a: G) {
                s.contains(a) and normal_subgroup_quotient_project(n, a) =
                    normal_subgroup_quotient_project(n, g)
            }
            let a: G satisfy {
                s.contains(a) and normal_subgroup_quotient_project(n, a) =
                    normal_subgroup_quotient_project(n, g)
            }
            normal_subgroup_rel_of_quotient_project_eq(n, a, g)
            normal_subgroup_rel(n, a, g)
            normal_subgroup_rel(n, a, g) = n.contains(a * g.inverse)
            n.contains(a * g.inverse)
            subgroup_subset(n, s) = forall(x: G) {
                n.contains(x) implies s.contains(x)
            }
            s.contains(a * g.inverse)
            subgroup_inv_mem(s, a * g.inverse)
            s.contains((a * g.inverse).inverse)
            subgroup_mul_mem(s, (a * g.inverse).inverse, a)
            inverse_mul(a, g.inverse)
            (a * g.inverse).inverse = g.inverse.inverse * a.inverse
            inverse_inverse(g)
            (a * g.inverse).inverse = g * a.inverse
            (g * a.inverse) * a = g * (a.inverse * a)
            a.inverse * a = G.1
            (g * a.inverse) * a = g * G.1
            g * G.1 = g
            (g * a.inverse) * a = g
            (a * g.inverse).inverse * a = g
            s.contains(g)
        }
        if s.contains(g) {
            projection_image_contains(n, s, normal_subgroup_quotient_project(n, g))
            projection_preimage_contains(n, projection_image_contains(n, s), g)
        }
        projection_preimage_contains(n, projection_image_contains(n, s), g) = s.contains(g)
    }
}

/// Membership of a quotient element in the image of the preimage of a
/// predicate `T` under the projection: the quotient element is the projection
/// of an element whose projection lies in `T`.
define projection_image_of_preimage_contains[G: Group](n: Subgroup[G], t: QuotientOver[G] -> Bool, q: QuotientOver[G]) -> Bool {
    exists(a: G) {
        projection_preimage_contains(n, t, a) and normal_subgroup_quotient_project(n, a) = q
    }
}

/// The right round trip: for a subgroup `T` of `G/N`, the image of the
/// preimage of `T` is `T` itself on every quotient element.
theorem projection_right_inverse[G: Group](n: Subgroup[G], t: QuotientOver[G] -> Bool, q: QuotientOver[G]) {
    q.qrel = normal_subgroup_quotient_relation(n) implies
    projection_image_of_preimage_contains(n, t, q) = t(q)
} by {
    if q.qrel = normal_subgroup_quotient_relation(n) {
        if projection_image_of_preimage_contains(n, t, q) {
            projection_image_of_preimage_contains(n, t, q) =
                exists(a: G) {
                    projection_preimage_contains(n, t, a) and
                    normal_subgroup_quotient_project(n, a) = q
                }
            let a: G satisfy {
                projection_preimage_contains(n, t, a) and
                normal_subgroup_quotient_project(n, a) = q
            }
            projection_preimage_contains(n, t, a) = t(normal_subgroup_quotient_project(n, a))
            t(normal_subgroup_quotient_project(n, a))
            t(q)
        }
        if t(q) {
            quotient_over_mk_surjective(q)
            let g: G satisfy {
                quotient_over_mk(q.qrel, g) = q
            }
            q.qrel = normal_subgroup_quotient_relation(n)
            quotient_over_mk(normal_subgroup_quotient_relation(n), g) = q
            normal_subgroup_quotient_project_eq_mk(n, g)
            normal_subgroup_quotient_project(n, g) = q
            projection_preimage_contains(n, t, g) = t(normal_subgroup_quotient_project(n, g))
            projection_preimage_contains(n, t, g)
            exists(a: G) {
                projection_preimage_contains(n, t, a) and
                normal_subgroup_quotient_project(n, a) = q
            }
            projection_image_of_preimage_contains(n, t, q)
        }
        projection_image_of_preimage_contains(n, t, q) = t(q)
    }
}
