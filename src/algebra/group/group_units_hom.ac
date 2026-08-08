/// Monoid homomorphisms and units: source-only first slice.

from algebra.one import One
from algebra.monoid.monoid import Monoid, MonoidHom, is_monoid_hom, monoid_hom_one, monoid_hom_mul,
    monoid_hom_ext, compose_monoid_hom, compose_monoid_hom_hom
from algebra.units import Unit, unit_one, unit_mul, unit_inverse, unit_ext, unit_ext_val,
    monoid_hom_unit, monoid_hom_unit_val, monoid_hom_unit_inv,
    monoid_hom_unit_one, monoid_hom_unit_mul, monoid_hom_unit_compose
from algebra.unit_group_instance import unit_typeclass_mul_eq_unit_mul, unit_typeclass_one_eq_unit_one
from data.basic.functions import compose, is_injective_fn, is_surjective_fn, is_bijection_fn,
    injective_fn_eq, bijection_fn_is_injective, bijection_fn_is_surjective

/// The function on units induced by a monoid homomorphism.
define units_map_fn[M: Monoid, N: Monoid](f: MonoidHom[M, N], u: Unit[M]) -> Unit[N] {
    monoid_hom_unit(f, u)
}

/// The induced function on units preserves the monoid structure.
theorem units_map_fn_is_monoid_hom[M: Monoid, N: Monoid](f: MonoidHom[M, N]) {
    is_monoid_hom(units_map_fn(f))
} by {
    unit_typeclass_one_eq_unit_one[M]
    One.1[Unit[M]] = unit_one[M]
    units_map_fn(f, One.1[Unit[M]]) = units_map_fn(f, unit_one[M])
    units_map_fn(f, unit_one[M]) = monoid_hom_unit(f, unit_one[M])
    monoid_hom_unit_one(f)
    monoid_hom_unit(f, unit_one[M]) = unit_one[N]
    unit_typeclass_one_eq_unit_one[N]
    One.1[Unit[N]] = unit_one[N]
    units_map_fn(f, One.1[Unit[M]]) = One.1[Unit[N]]

    forall(u: Unit[M], v: Unit[M]) {
        unit_typeclass_mul_eq_unit_mul(u, v)
        u * v = unit_mul(u, v)
        units_map_fn(f, u * v) = units_map_fn(f, unit_mul(u, v))
        units_map_fn(f, unit_mul(u, v)) = monoid_hom_unit(f, unit_mul(u, v))
        monoid_hom_unit_mul(f, u, v)
        monoid_hom_unit(f, unit_mul(u, v)) = unit_mul(monoid_hom_unit(f, u), monoid_hom_unit(f, v))
        units_map_fn(f, u) = monoid_hom_unit(f, u)
        units_map_fn(f, v) = monoid_hom_unit(f, v)
        units_map_fn(f, u) * units_map_fn(f, v) =
            monoid_hom_unit(f, u) * monoid_hom_unit(f, v)
        unit_typeclass_mul_eq_unit_mul(monoid_hom_unit(f, u), monoid_hom_unit(f, v))
        monoid_hom_unit(f, u) * monoid_hom_unit(f, v) =
            unit_mul(monoid_hom_unit(f, u), monoid_hom_unit(f, v))
        units_map_fn(f, u * v) = units_map_fn(f, u) * units_map_fn(f, v)
    }
    is_monoid_hom(units_map_fn(f))
}

/// The monoid homomorphism on units induced by a monoid homomorphism.
let units_map[M: Monoid, N: Monoid](f: MonoidHom[M, N]) -> result: MonoidHom[Unit[M], Unit[N]] satisfy {
    MonoidHom.new(units_map_fn(f)) = Option.some(result)
} by {
    units_map_fn_is_monoid_hom(f)
}

/// The underlying function of `units_map` is `units_map_fn`.
theorem units_map_hom[M: Monoid, N: Monoid](f: MonoidHom[M, N]) {
    units_map(f).hom = units_map_fn(f)
}

/// The value of a mapped unit is the image of the value.
theorem coe_map[M: Monoid, N: Monoid](f: MonoidHom[M, N], u: Unit[M]) {
    units_map(f).hom(u).val = f.hom(u.val)
} by {
    units_map_hom(f)
    units_map(f).hom = units_map_fn(f)
    units_map(f).hom(u) = units_map_fn(f, u)
    units_map_fn(f, u) = monoid_hom_unit(f, u)
    monoid_hom_unit_val(f, u)
}

/// The inverse component of a mapped unit is the image of the inverse component.
theorem coe_map_inv[M: Monoid, N: Monoid](f: MonoidHom[M, N], u: Unit[M]) {
    units_map(f).hom(u).inv = f.hom(u.inv)
} by {
    units_map_hom(f)
    units_map(f).hom = units_map_fn(f)
    units_map(f).hom(u) = units_map_fn(f, u)
    units_map_fn(f, u) = monoid_hom_unit(f, u)
    monoid_hom_unit_inv(f, u)
}
