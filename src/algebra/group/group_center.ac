/// First-slice wrappers for multiplicative central elements.
///
/// This mirrors the small `IsMulCentral` structure from Mathlib's
/// `Mathlib/Algebra/Group/Center.lean`, using only the local `Mul` and
/// `Semigroup` surface.

from algebra.mul import Mul
from algebra.semigroup import Semigroup

/// Two multiplicative elements commute.
define mul_commute[M: Mul](a: M, b: M) -> Bool {
    a * b = b * a
}

/// Conditions for an element to be multiplicatively central.
///
/// This is the predicate-shaped analogue of Mathlib `IsMulCentral`: the
/// element commutes with every element, and the two associativity clauses hold
/// when it appears on the left or right.
define is_mul_central[M: Mul](z: M) -> Bool {
    forall(a: M) {
        z * a = a * z
    } and forall(b: M, c: M) {
        z * (b * c) = (z * b) * c
    } and forall(a: M, b: M) {
        (a * b) * z = a * (b * z)
    }
}

/// Unfolding the local commutation predicate is just equality of products.
theorem mul_commute_eq[M: Mul](a: M, b: M) {
    mul_commute(a, b) = (a * b = b * a)
}

/// A central element commutes with every element.
theorem is_mul_central_comm[M: Mul](z: M, a: M) {
    is_mul_central(z) implies z * a = a * z
} by {
    if is_mul_central(z) {
        forall(x: M) {
            z * x = x * z
        }
    }
}

/// A central element satisfies the left associativity clause.
theorem is_mul_central_left_assoc[M: Mul](z: M, b: M, c: M) {
    is_mul_central(z) implies z * (b * c) = (z * b) * c
} by {
    if is_mul_central(z) {
        forall(x: M, y: M) {
            z * (x * y) = (z * x) * y
        }
    }
}

/// A central element satisfies the right associativity clause.
theorem is_mul_central_right_assoc[M: Mul](z: M, a: M, b: M) {
    is_mul_central(z) implies (a * b) * z = a * (b * z)
} by {
    if is_mul_central(z) {
        forall(x: M, y: M) {
            (x * y) * z = x * (y * z)
        }
    }
}

/// The predicate unfolds to the three Mathlib-style fields.
theorem is_mul_central_eq[M: Mul](z: M) {
    is_mul_central(z) = (
        forall(a: M) {
            z * a = a * z
        } and forall(b: M, c: M) {
            z * (b * c) = (z * b) * c
        } and forall(a: M, b: M) {
            (a * b) * z = a * (b * z)
        }
    )
} by {
    is_mul_central(z) = (
        forall(a: M) {
            z * a = a * z
        } and forall(b: M, c: M) {
            z * (b * c) = (z * b) * c
        } and forall(a: M, b: M) {
            (a * b) * z = a * (b * z)
        }
    )
}

/// Introduction wrapper for the centrality predicate.
theorem is_mul_central_intro[M: Mul](z: M) {
    (forall(a: M) {
        z * a = a * z
    } and forall(b: M, c: M) {
        z * (b * c) = (z * b) * c
    } and forall(a: M, b: M) {
        (a * b) * z = a * (b * z)
    }) implies is_mul_central(z)
} by {
    if forall(a: M) {
        z * a = a * z
    } and forall(b: M, c: M) {
        z * (b * c) = (z * b) * c
    } and forall(a: M, b: M) {
        (a * b) * z = a * (b * z)
    } {
        is_mul_central(z)
    }
}

/// In a semigroup, the left associativity clause follows from semigroup associativity.
theorem semigroup_mul_central_left_assoc[M: Semigroup](z: M, b: M, c: M) {
    z * (b * c) = (z * b) * c
} by {
    M.mul_associative(z, b, c)
}

/// In a semigroup, the right associativity clause follows from semigroup associativity.
theorem semigroup_mul_central_right_assoc[M: Semigroup](z: M, a: M, b: M) {
    (a * b) * z = a * (b * z)
} by {
    M.mul_associative(a, b, z)
}
