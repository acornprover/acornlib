from algebra.group import Group, inverse_mul, inverse_left, inverse_inverse
from algebra.group_action import MulAction, is_mul_action, action_identity_constraint,
    action_mul_constraint, orbit, orbit_contains_self, orbit_contains_witness,
    orbit_contains_action, stabilizer, stabilizer_contains_eq,
    stabilizer_contains_of_fixes, stabilizer_fixes
from pair import curry, uncurry

/// Conjugation of x by g.
define conjugation_act[G: Group](g: G, x: G) -> G {
    g * x * g.inverse
}

/// Conjugation by the identity is trivial.
theorem conjugation_act_one[G: Group](x: G) {
    conjugation_act(G.1, x) = x
} by {
    G.1.inverse = G.1
}

/// Conjugation composes with group multiplication.
theorem conjugation_act_mul[G: Group](g: G, h: G, x: G) {
    conjugation_act(g * h, x) = conjugation_act(g, conjugation_act(h, x))
} by {
    inverse_mul(g, h)
    conjugation_act(g * h, x) = (g * h) * x * (h.inverse * g.inverse)
    conjugation_act(g, conjugation_act(h, x)) = g * (h * x * h.inverse) * g.inverse
}

/// Conjugation satisfies the action identity constraint.
theorem conjugation_action_identity_constraint[G: Group] {
    action_identity_constraint(conjugation_act[G])
} by {
    forall(x: G) {
        conjugation_act_one(x)
        conjugation_act[G](G.1, x) = x
    }
}

/// Conjugation satisfies the action multiplication constraint.
theorem conjugation_action_mul_constraint[G: Group] {
    action_mul_constraint(conjugation_act[G])
} by {
    forall(g: G, h: G, x: G) {
        conjugation_act_mul(g, h, x)
    }
}

/// Conjugation is a left action of G on itself.
theorem conjugation_is_mul_action[G: Group] {
    is_mul_action(conjugation_act[G])
} by {
    conjugation_action_identity_constraint[G]
    conjugation_action_mul_constraint[G]
}

/// The bundled conjugation action of G on itself.
let conjugation_action[G: Group]: MulAction[G, G] satisfy {
    MulAction.new(conjugation_act[G]) = Option.some(conjugation_action)
}

/// The action map of `conjugation_action` is `conjugation_act`.
theorem conjugation_action_act[G: Group](g: G, x: G) {
    conjugation_action[G].act(g, x) = g * x * g.inverse
}

/// Conjugation fixes an element if and only if it commutes with the conjugating element.
theorem conjugation_act_fixed_iff_commutes[G: Group](g: G, x: G) {
    conjugation_act(g, x) = x iff g * x = x * g
} by {
    if conjugation_act(g, x) = x {
        g * x * (g.inverse * g) = x * g
        g * x = x * g
    }
    if g * x = x * g {
        x * g * g.inverse = x * (g * g.inverse)
        conjugation_act(g, x) = x
    }
}

/// The identity element is fixed by conjugation by every group element.
theorem conjugation_act_one_value[G: Group](g: G) {
    conjugation_act(g, G.1) = G.1
} by {
    g * g.inverse = G.1
}

/// Conjugation preserves the group operation in the second argument.
theorem conjugation_act_mul_value[G: Group](g: G, x: G, y: G) {
    conjugation_act(g, x * y) = conjugation_act(g, x) * conjugation_act(g, y)
} by {
    g.inverse * g = G.1
    conjugation_act(g, x) * conjugation_act(g, y) = (g * x * g.inverse) * (g * y * g.inverse)
    (g * x * g.inverse) * (g * y * g.inverse) = g * x * (g.inverse * g) * y * g.inverse
    g * x * G.1 * y * g.inverse = g * x * y * g.inverse
}

/// Conjugation preserves inverses.
theorem conjugation_act_inverse[G: Group](g: G, x: G) {
    conjugation_act(g, x.inverse) = conjugation_act(g, x).inverse
} by {
    inverse_mul(g, x)
    inverse_mul(g * x, g.inverse)
    (g * x * g.inverse).inverse = g * x.inverse * g.inverse
}

/// Every element is fixed by conjugation by itself.
theorem conjugation_act_self[G: Group](x: G) {
    conjugation_act(x, x) = x
} by {
    x * x = x * x
    conjugation_act_fixed_iff_commutes(x, x)
}

/// Conjugating by an inverse is the inverse operation of conjugating by the element.
theorem conjugation_act_inverse_arg[G: Group](g: G, x: G) {
    conjugation_act(g.inverse, conjugation_act(g, x)) = x
} by {
    conjugation_act_mul(g.inverse, g, x)
    conjugation_act_one(x)
}

/// Conjugation is reversible by conjugating with the inverse on the other side.
theorem conjugation_act_arg_inverse[G: Group](g: G, x: G) {
    conjugation_act(g, conjugation_act(g.inverse, x)) = x
} by {
    conjugation_act_mul(g, g.inverse, x)
    conjugation_act_one(x)
}

/// Conjugation by g is injective in its second argument.
theorem conjugation_act_injective[G: Group](g: G, x: G, y: G) {
    conjugation_act(g, x) = conjugation_act(g, y) implies x = y
} by {
    if conjugation_act(g, x) = conjugation_act(g, y) {
        conjugation_act_inverse_arg(g, x)
        conjugation_act_inverse_arg(g, y)
        conjugation_act(g.inverse, conjugation_act(g, x)) = conjugation_act(g.inverse, conjugation_act(g, y))
    }
}

/// Conjugation by g is surjective in its second argument.
theorem conjugation_act_surjective[G: Group](g: G, y: G) {
    exists(x: G) { conjugation_act(g, x) = y }
} by {
    conjugation_act_arg_inverse(g, y)
}

/// Conjugation by an inverse, expressed without nested inverses.
theorem conjugation_act_inverse_arg_eq[G: Group](g: G, x: G) {
    conjugation_act(g.inverse, x) = g.inverse * x * g
} by {
}

/// Conjugation by an element fixes its own inverse.
theorem conjugation_act_self_inverse[G: Group](x: G) {
    conjugation_act(x, x.inverse) = x.inverse
} by {
    conjugation_act_self(x)
    conjugation_act_inverse(x, x)
}

/// Membership in a conjugation orbit is exactly conjugacy.
theorem conjugation_action_orbit_contains_iff[G: Group](x: G, y: G) {
    orbit(conjugation_action[G], x).contains(y) iff exists(g: G) {
        g * x * g.inverse = y
    }
} by {
    if orbit(conjugation_action[G], x).contains(y) {
        orbit_contains_witness(conjugation_action[G], x, y)
        let g: G satisfy {
            y = conjugation_action[G].act(g, x)
        }
        conjugation_action_act(g, x)
        g * x * g.inverse = y
    }
    if exists(g: G) { g * x * g.inverse = y } {
        let g: G satisfy {
            g * x * g.inverse = y
        }
        conjugation_action_act(g, x)
        orbit_contains_action(conjugation_action[G], x, g)
        orbit(conjugation_action[G], x).contains(y)
    }
}

/// Every element is in its own conjugation orbit.
theorem conjugation_action_orbit_contains_self[G: Group](x: G) {
    orbit(conjugation_action[G], x).contains(x)
} by {
    orbit_contains_self(conjugation_action[G], x)
}

/// The stabilizer of x in the conjugation action consists of elements commuting with x.
theorem conjugation_action_stabilizer_contains_iff[G: Group](x: G, g: G) {
    stabilizer(conjugation_action[G], x).contains(g) iff g * x = x * g
} by {
    stabilizer_contains_eq(conjugation_action[G], x, g)
    conjugation_action_act(g, x)
    conjugation_act_fixed_iff_commutes(g, x)
    if stabilizer(conjugation_action[G], x).contains(g) {
        g * x = x * g
    }
    if g * x = x * g {
        conjugation_act(g, x) = x
        conjugation_action[G].act(g, x) = x
        stabilizer_contains_of_fixes(conjugation_action[G], x, g)
    }
}

/// An element commuting with x belongs to the conjugation stabilizer of x.
theorem conjugation_action_stabilizer_of_commutes[G: Group](x: G, g: G) {
    g * x = x * g implies stabilizer(conjugation_action[G], x).contains(g)
} by {
    conjugation_action_stabilizer_contains_iff(x, g)
}

/// A member of the conjugation stabilizer commutes with x.
theorem conjugation_action_stabilizer_commutes[G: Group](x: G, g: G) {
    stabilizer(conjugation_action[G], x).contains(g) implies g * x = x * g
} by {
    conjugation_action_stabilizer_contains_iff(x, g)
}

/// The identity element belongs to every conjugation stabilizer.
theorem conjugation_action_stabilizer_contains_one[G: Group](x: G) {
    stabilizer(conjugation_action[G], x).contains(G.1)
} by {
    conjugation_action_stabilizer_of_commutes(x, G.1)
}

/// Every element belongs to its own conjugation stabilizer.
theorem conjugation_action_stabilizer_contains_self[G: Group](x: G) {
    stabilizer(conjugation_action[G], x).contains(x)
} by {
    x * x = x * x
    conjugation_action_stabilizer_of_commutes(x, x)
}

/// Every element belongs to the conjugation stabilizer of its own inverse.
theorem conjugation_action_stabilizer_contains_inverse[G: Group](x: G) {
    stabilizer(conjugation_action[G], x).contains(x.inverse)
} by {
    conjugation_action_stabilizer_of_commutes(x, x.inverse)
}

/// The conjugation stabilizer is closed under multiplication.
theorem conjugation_action_stabilizer_mul[G: Group](x: G, g: G, h: G) {
    stabilizer(conjugation_action[G], x).contains(g) and
        stabilizer(conjugation_action[G], x).contains(h) implies
        stabilizer(conjugation_action[G], x).contains(g * h)
} by {
    if stabilizer(conjugation_action[G], x).contains(g) and
        stabilizer(conjugation_action[G], x).contains(h) {
        conjugation_action_stabilizer_commutes(x, g)
        conjugation_action_stabilizer_commutes(x, h)
        (g * x) * h = (x * g) * h
        (g * h) * x = x * (g * h)
        conjugation_action_stabilizer_of_commutes(x, g * h)
    }
}

/// The conjugation stabilizer is closed under taking inverses.
theorem conjugation_action_stabilizer_inverse[G: Group](x: G, g: G) {
    stabilizer(conjugation_action[G], x).contains(g) implies
        stabilizer(conjugation_action[G], x).contains(g.inverse)
} by {
    if stabilizer(conjugation_action[G], x).contains(g) {
        conjugation_action_stabilizer_commutes(x, g)
        g.inverse * (g * x) = x
        g.inverse * (x * g) * g.inverse = g.inverse * x * (g * g.inverse)
        g.inverse * x = x * g.inverse
        conjugation_action_stabilizer_of_commutes(x, g.inverse)
    }
}

/// Conjugacy is a symmetric relation: orbits are mutually inclusive.
theorem conjugation_action_orbit_symmetric[G: Group](x: G, y: G) {
    orbit(conjugation_action[G], x).contains(y) implies
        orbit(conjugation_action[G], y).contains(x)
} by {
    if orbit(conjugation_action[G], x).contains(y) {
        conjugation_action_orbit_contains_iff(x, y)
        let g: G satisfy {
            g * x * g.inverse = y
        }
        // From y = g x g.inverse derive x = g.inverse y g.
        g.inverse * (g * x * g.inverse) * g = (g.inverse * g) * x * (g.inverse * g)
        g.inverse * y * g.inverse.inverse = x
        exists(h: G) { h * y * h.inverse = x }
        conjugation_action_orbit_contains_iff(y, x)
    }
}

/// Conjugacy via g witnesses conjugacy via g.inverse in the reverse direction.
theorem conjugation_act_inverse_witness[G: Group](g: G, x: G) {
    conjugation_act(g.inverse, conjugation_act(g, x)) = x
} by {
    conjugation_act_inverse_arg(g, x)
}

/// The conjugation orbit of the identity contains only the identity.
theorem conjugation_action_orbit_one_eq[G: Group](y: G) {
    orbit(conjugation_action[G], G.1).contains(y) iff y = G.1
} by {
    if orbit(conjugation_action[G], G.1).contains(y) {
        conjugation_action_orbit_contains_iff(G.1, y)
        let g: G satisfy {
            g * G.1 * g.inverse = y
        }
        y = G.1
    }
    if y = G.1 {
        conjugation_action_orbit_contains_self(G.1)
        orbit(conjugation_action[G], G.1).contains(y)
    }
}

/// The conjugate of x by its inverse is x itself.
theorem conjugation_act_self_inverse_arg[G: Group](x: G) {
    conjugation_act(x.inverse, x) = x
} by {
    conjugation_act_inverse_arg_eq(x, x)
}

/// Two conjugation orbits relating x to y also relate y to x via the inverse witness.
theorem conjugation_action_conjugate_inverse_witness[G: Group](g: G, x: G, y: G) {
    g * x * g.inverse = y implies g.inverse * y * g = x
} by {
    if g * x * g.inverse = y {
        g.inverse * (g * x * g.inverse) * g = (g.inverse * g) * x * (g.inverse * g)
        g.inverse * y * g = x
    }
}

/// Membership of `g * h * g.inverse` in the conjugation orbit of h.
theorem conjugation_action_orbit_contains_conjugate[G: Group](g: G, h: G) {
    orbit(conjugation_action[G], h).contains(g * h * g.inverse)
} by {
    conjugation_action_act(g, h)
    orbit_contains_action(conjugation_action[G], h, g)
}
