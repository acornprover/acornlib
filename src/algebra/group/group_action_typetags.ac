/// Additive and multiplicative type tags for group actions.
///
/// This module mirrors the small `Mathlib/Algebra/Group/Action/TypeTags.lean`
/// bridge in the Acorn library style.  It introduces lightweight wrappers that
/// reinterpret a multiplicative group as an additive group and an additive group
/// as a multiplicative group, then transports actions across those wrappers.

from algebra.group import Group
from algebra.comm_group import CommGroup
from algebra.add import Add
from algebra.zero import Zero
from algebra.neg import Neg
from algebra.mul import Mul
from algebra.one import One
from algebra.add_semigroup import AddSemigroup
from algebra.add_comm_semigroup import AddCommSemigroup
from algebra.add_monoid import AddMonoid
from algebra.add_comm_monoid import AddCommMonoid
from algebra.add_group import AddGroup
from algebra.add_comm_group import AddCommGroup
from algebra.semigroup import Semigroup
from algebra.comm_semigroup import CommSemigroup
from algebra.monoid.monoid import Monoid
from algebra.comm_monoid import CommMonoid
from algebra.group_action import MulAction, is_mul_action, action_identity_constraint,
    action_mul_constraint, mul_action_one, mul_action_mul

/// Reinterpret a multiplicative carrier as an additive carrier.
structure Additive[M] {
    /// The underlying multiplicative value.
    val: M
}

/// Reinterpret an additive carrier as a multiplicative carrier.
structure Multiplicative[A] {
    /// The underlying additive value.
    val: A
}

/// Wrap a multiplicative value as an additive tagged value.
define additive_of_mul[M](x: M) -> Additive[M] {
    Additive.new(x)
}

/// Forget the additive tag and recover the multiplicative value.
define additive_to_mul[M](x: Additive[M]) -> M {
    x.val
}

/// Wrap an additive value as a multiplicative tagged value.
define multiplicative_of_add[A](x: A) -> Multiplicative[A] {
    Multiplicative.new(x)
}

/// Forget the multiplicative tag and recover the additive value.
define multiplicative_to_add[A](x: Multiplicative[A]) -> A {
    x.val
}

/// Additive-tag extensionality from equality of underlying values.
theorem additive_ext[M](x: Additive[M], y: Additive[M]) {
    x.val = y.val implies x = y
} by {
    if x.val = y.val {
        Additive.new(x.val) = x
        Additive.new(y.val) = y
        Additive.new(x.val) = Additive.new(y.val)
        x = y
    }
}

/// Multiplicative-tag extensionality from equality of underlying values.
theorem multiplicative_ext[A](x: Multiplicative[A], y: Multiplicative[A]) {
    x.val = y.val implies x = y
} by {
    if x.val = y.val {
        Multiplicative.new(x.val) = x
        Multiplicative.new(y.val) = y
        Multiplicative.new(x.val) = Multiplicative.new(y.val)
        x = y
    }
}

/// Unwrapping a freshly additive-tagged value returns the original value.
theorem additive_to_of_mul[M](x: M) {
    additive_to_mul(additive_of_mul(x)) = x
} by {
    additive_of_mul(x) = Additive.new(x)
    additive_to_mul(additive_of_mul(x)) = additive_of_mul(x).val
    Additive.new(x).val = x
}

/// Re-wrapping an unwrapped additive-tagged value returns the original wrapper.
theorem additive_of_to_mul[M](x: Additive[M]) {
    additive_of_mul(additive_to_mul(x)) = x
} by {
    additive_to_mul(x) = x.val
    additive_of_mul(additive_to_mul(x)) = Additive.new(x.val)
    Additive.new(x.val) = x
}

/// Unwrapping a freshly multiplicative-tagged value returns the original value.
theorem multiplicative_to_of_add[A](x: A) {
    multiplicative_to_add(multiplicative_of_add(x)) = x
} by {
    multiplicative_of_add(x) = Multiplicative.new(x)
    multiplicative_to_add(multiplicative_of_add(x)) = multiplicative_of_add(x).val
    Multiplicative.new(x).val = x
}

/// Re-wrapping an unwrapped multiplicative-tagged value returns the original wrapper.
theorem multiplicative_of_to_add[A](x: Multiplicative[A]) {
    multiplicative_of_add(multiplicative_to_add(x)) = x
} by {
    multiplicative_to_add(x) = x.val
    multiplicative_of_add(multiplicative_to_add(x)) = Multiplicative.new(x.val)
    Multiplicative.new(x.val) = x
}

/// Addition on an additive tag is multiplication of the underlying values.
attributes Additive[M: Group] {
    define add(self, other: Additive[M]) -> Additive[M] {
        Additive.new(self.val * other.val)
    }

    /// The additive zero is the multiplicative identity.
    let zero: Additive[M] = Additive.new(M.1)

    /// Additive negation is multiplicative inversion.
    define neg(self) -> Additive[M] {
        Additive.new(self.val.inverse)
    }
}

/// Multiplication on a multiplicative tag is addition of the underlying values.
attributes Multiplicative[A: AddGroup] {
    define mul(self, other: Multiplicative[A]) -> Multiplicative[A] {
        Multiplicative.new(self.val + other.val)
    }

    /// The multiplicative identity is the additive zero.
    let one: Multiplicative[A] = Multiplicative.new(A.0)

    /// Multiplicative inversion is additive negation.
    define inverse(self) -> Multiplicative[A] {
        Multiplicative.new(-self.val)
    }
}

/// Additive tags have typeclass addition.
instance Additive[M: Group]: Add {
    let add = Additive[M].add
}

/// Additive tags have typeclass zero.
instance Additive[M: Group]: Zero {
    let 0 = Additive[M].zero
}

/// Additive tags have typeclass negation.
instance Additive[M: Group]: Neg {
    let neg = Additive[M].neg
}

/// Multiplicative tags have typeclass multiplication.
instance Multiplicative[A: AddGroup]: Mul {
    let mul = Multiplicative[A].mul
}

/// Multiplicative tags have typeclass one.
instance Multiplicative[A: AddGroup]: One {
    let 1 = Multiplicative[A].one
}

/// The underlying value of a sum in an additive tag is the product of values.
theorem additive_val_add[M: Group](x: Additive[M], y: Additive[M]) {
    (x + y).val = x.val * y.val
} by {
    x + y = Additive.new(x.val * y.val)
    Additive.new(x.val * y.val).val = x.val * y.val
}

/// The underlying value of additive zero is the multiplicative identity.
theorem additive_val_zero[M: Group] {
    Zero.0[Additive[M]].val = M.1
} by {
    Zero.0[Additive[M]] = Additive[M].zero
    Additive[M].zero = Additive.new(M.1)
    Additive.new(M.1).val = M.1
}

/// The underlying value of additive negation is the multiplicative inverse.
theorem additive_val_neg[M: Group](x: Additive[M]) {
    (-x).val = x.val.inverse
} by {
    -x = Additive.new(x.val.inverse)
    Additive.new(x.val.inverse).val = x.val.inverse
}

/// Wrapping a product agrees with adding wrapped values.
theorem additive_of_mul_mul[M: Group](x: M, y: M) {
    additive_of_mul(x * y) = additive_of_mul(x) + additive_of_mul(y)
} by {
    additive_of_mul(x * y).val = x * y
    additive_val_add(additive_of_mul(x), additive_of_mul(y))
    additive_of_mul(x).val = x
    additive_of_mul(y).val = y
    (additive_of_mul(x) + additive_of_mul(y)).val = x * y
    additive_of_mul(x * y).val = (additive_of_mul(x) + additive_of_mul(y)).val
    additive_ext(additive_of_mul(x * y), additive_of_mul(x) + additive_of_mul(y))
}

/// Wrapping one gives additive zero.
theorem additive_of_mul_one[M: Group] {
    additive_of_mul(M.1) = Zero.0[Additive[M]]
} by {
    additive_of_mul(M.1).val = M.1
    additive_val_zero[M]
    additive_of_mul(M.1).val = Zero.0[Additive[M]].val
    additive_ext(additive_of_mul(M.1), Zero.0[Additive[M]])
}

/// Wrapping an inverse gives additive negation.
theorem additive_of_mul_inverse[M: Group](x: M) {
    additive_of_mul(x.inverse) = -additive_of_mul(x)
} by {
    additive_of_mul(x.inverse).val = x.inverse
    additive_val_neg(additive_of_mul(x))
    additive_of_mul(x).val = x
    (-additive_of_mul(x)).val = x.inverse
    additive_of_mul(x.inverse).val = (-additive_of_mul(x)).val
    additive_ext(additive_of_mul(x.inverse), -additive_of_mul(x))
}

/// Additive-tag addition is associative.
theorem additive_add_assoc[M: Group](x: Additive[M], y: Additive[M], z: Additive[M]) {
    x + (y + z) = (x + y) + z
} by {
    additive_val_add(y, z)
    (y + z).val = y.val * z.val
    additive_val_add(x, y + z)
    (x + (y + z)).val = x.val * (y + z).val
    (x + (y + z)).val = x.val * (y.val * z.val)

    additive_val_add(x, y)
    (x + y).val = x.val * y.val
    additive_val_add(x + y, z)
    ((x + y) + z).val = (x + y).val * z.val
    ((x + y) + z).val = (x.val * y.val) * z.val

    x.val * (y.val * z.val) = (x.val * y.val) * z.val
    (x + (y + z)).val = ((x + y) + z).val
    additive_ext(x + (y + z), (x + y) + z)
}

/// Additive tags satisfy the additive-semigroup law in primitive form.
theorem additive_add_semigroup_instance_law[M: Group](x: Additive[M], y: Additive[M], z: Additive[M]) {
    Add.add(x, Add.add(y, z)) = Add.add(Add.add(x, y), z)
} by {
    additive_add_assoc(x, y, z)
}

/// Additive tags over a group form an additive semigroup.
instance Additive[M: Group]: AddSemigroup

/// Additive zero is a right identity.
theorem additive_add_zero_right[M: Group](x: Additive[M]) {
    x + Zero.0[Additive[M]] = x
} by {
    additive_val_add(x, Zero.0[Additive[M]])
    additive_val_zero[M]
    (x + Zero.0[Additive[M]]).val = x.val * Zero.0[Additive[M]].val
    (x + Zero.0[Additive[M]]).val = x.val * M.1
    x.val * M.1 = x.val
    (x + Zero.0[Additive[M]]).val = x.val
    additive_ext(x + Zero.0[Additive[M]], x)
}

/// Additive zero is a left identity.
theorem additive_add_zero_left[M: Group](x: Additive[M]) {
    Zero.0[Additive[M]] + x = x
} by {
    additive_val_add(Zero.0[Additive[M]], x)
    additive_val_zero[M]
    (Zero.0[Additive[M]] + x).val = Zero.0[Additive[M]].val * x.val
    (Zero.0[Additive[M]] + x).val = M.1 * x.val
    M.1 * x.val = x.val
    (Zero.0[Additive[M]] + x).val = x.val
    additive_ext(Zero.0[Additive[M]] + x, x)
}

/// Additive tags satisfy the right identity law in primitive form.
theorem additive_add_monoid_right_law[M: Group] {
    forall(x: Additive[M]) {
        Add.add(x, Zero.0[Additive[M]]) = x
    }
} by {
    forall(x: Additive[M]) {
        additive_add_zero_right(x)
    }
}

/// Additive tags satisfy the left identity law in primitive form.
theorem additive_add_monoid_left_law[M: Group] {
    forall(x: Additive[M]) {
        Add.add(Zero.0[Additive[M]], x) = x
    }
} by {
    forall(x: Additive[M]) {
        additive_add_zero_left(x)
    }
}

/// Additive tags over a group form an additive monoid.
instance Additive[M: Group]: AddMonoid

/// Additive negation is a right inverse.
theorem additive_add_neg_right[M: Group](x: Additive[M]) {
    x + -x = Zero.0[Additive[M]]
} by {
    additive_val_add(x, -x)
    additive_val_neg(x)
    (x + -x).val = x.val * (-x).val
    (x + -x).val = x.val * x.val.inverse
    x.val * x.val.inverse = M.1
    (x + -x).val = M.1
    additive_val_zero[M]
    Zero.0[Additive[M]].val = M.1
    (x + -x).val = Zero.0[Additive[M]].val
    additive_ext(x + -x, Zero.0[Additive[M]])
}

/// Additive tags satisfy the additive-group inverse law in primitive form.
theorem additive_add_group_inverse_law[M: Group](x: Additive[M]) {
    Add.add(x, Neg.neg(x)) = Zero.0[Additive[M]]
} by {
    additive_add_neg_right(x)
}

/// Additive tags over a group form an additive group.
instance Additive[M: Group]: AddGroup

/// Addition in an additive tag is commutative when the original group is commutative.
theorem additive_add_comm[M: CommGroup](x: Additive[M], y: Additive[M]) {
    x + y = y + x
} by {
    additive_val_add(x, y)
    additive_val_add(y, x)
    (x + y).val = x.val * y.val
    (y + x).val = y.val * x.val
    x.val * y.val = y.val * x.val
    (x + y).val = (y + x).val
    additive_ext(x + y, y + x)
}

/// Additive tags satisfy the additive-commutative-semigroup law in primitive form.
theorem additive_add_comm_semigroup_instance_law[M: CommGroup](x: Additive[M], y: Additive[M]) {
    Add.add(x, y) = Add.add(y, x)
} by {
    additive_add_comm(x, y)
}

/// Additive tags over a commutative group form an additive commutative semigroup.
instance Additive[M: CommGroup]: AddCommSemigroup

/// Additive tags over a commutative group form an additive commutative monoid.
instance Additive[M: CommGroup]: AddCommMonoid

/// Additive tags over a commutative group form an additive commutative group.
instance Additive[M: CommGroup]: AddCommGroup

/// The underlying value of a product in a multiplicative tag is the sum of values.
theorem multiplicative_val_mul[A: AddGroup](x: Multiplicative[A], y: Multiplicative[A]) {
    (x * y).val = x.val + y.val
} by {
    x * y = Multiplicative.new(x.val + y.val)
    Multiplicative.new(x.val + y.val).val = x.val + y.val
}

/// The underlying value of multiplicative one is additive zero.
theorem multiplicative_val_one[A: AddGroup] {
    One.1[Multiplicative[A]].val = A.0
} by {
    One.1[Multiplicative[A]] = Multiplicative[A].one
    Multiplicative[A].one = Multiplicative.new(A.0)
    Multiplicative.new(A.0).val = A.0
}

/// The underlying value of multiplicative inversion is additive negation.
theorem multiplicative_val_inverse[A: AddGroup](x: Multiplicative[A]) {
    x.inverse.val = -x.val
} by {
    x.inverse = Multiplicative.new(-x.val)
    Multiplicative.new(-x.val).val = -x.val
}

/// Wrapping an additive sum agrees with multiplying wrapped values.
theorem multiplicative_of_add_add[A: AddGroup](x: A, y: A) {
    multiplicative_of_add(x + y) = multiplicative_of_add(x) * multiplicative_of_add(y)
} by {
    multiplicative_of_add(x + y).val = x + y
    multiplicative_val_mul(multiplicative_of_add(x), multiplicative_of_add(y))
    multiplicative_of_add(x).val = x
    multiplicative_of_add(y).val = y
    (multiplicative_of_add(x) * multiplicative_of_add(y)).val = x + y
    multiplicative_of_add(x + y).val = (multiplicative_of_add(x) * multiplicative_of_add(y)).val
    multiplicative_ext(multiplicative_of_add(x + y), multiplicative_of_add(x) * multiplicative_of_add(y))
}

/// Wrapping additive zero gives multiplicative one.
theorem multiplicative_of_add_zero[A: AddGroup] {
    multiplicative_of_add(A.0) = One.1[Multiplicative[A]]
} by {
    multiplicative_of_add(A.0).val = A.0
    multiplicative_val_one[A]
    multiplicative_of_add(A.0).val = One.1[Multiplicative[A]].val
    multiplicative_ext(multiplicative_of_add(A.0), One.1[Multiplicative[A]])
}

/// Wrapping additive negation gives multiplicative inversion.
theorem multiplicative_of_add_neg[A: AddGroup](x: A) {
    multiplicative_of_add(-x) = multiplicative_of_add(x).inverse
} by {
    multiplicative_of_add(-x).val = -x
    multiplicative_val_inverse(multiplicative_of_add(x))
    multiplicative_of_add(x).val = x
    multiplicative_of_add(x).inverse.val = -x
    multiplicative_of_add(-x).val = multiplicative_of_add(x).inverse.val
    multiplicative_ext(multiplicative_of_add(-x), multiplicative_of_add(x).inverse)
}

/// Multiplicative-tag multiplication is associative.
theorem multiplicative_mul_assoc[A: AddGroup](x: Multiplicative[A], y: Multiplicative[A], z: Multiplicative[A]) {
    x * (y * z) = (x * y) * z
} by {
    multiplicative_val_mul(y, z)
    (y * z).val = y.val + z.val
    multiplicative_val_mul(x, y * z)
    (x * (y * z)).val = x.val + (y * z).val
    (x * (y * z)).val = x.val + (y.val + z.val)

    multiplicative_val_mul(x, y)
    (x * y).val = x.val + y.val
    multiplicative_val_mul(x * y, z)
    ((x * y) * z).val = (x * y).val + z.val
    ((x * y) * z).val = (x.val + y.val) + z.val

    x.val + (y.val + z.val) = (x.val + y.val) + z.val
    (x * (y * z)).val = ((x * y) * z).val
    multiplicative_ext(x * (y * z), (x * y) * z)
}

/// Multiplicative tags satisfy the semigroup law in primitive form.
theorem multiplicative_semigroup_instance_law[A: AddGroup](x: Multiplicative[A], y: Multiplicative[A], z: Multiplicative[A]) {
    Mul.mul(x, Mul.mul(y, z)) = Mul.mul(Mul.mul(x, y), z)
} by {
    multiplicative_mul_assoc(x, y, z)
}

/// Multiplicative tags over an additive group form a semigroup.
instance Multiplicative[A: AddGroup]: Semigroup

/// Multiplicative one is a right identity.
theorem multiplicative_mul_one_right[A: AddGroup](x: Multiplicative[A]) {
    x * One.1[Multiplicative[A]] = x
} by {
    multiplicative_val_mul(x, One.1[Multiplicative[A]])
    multiplicative_val_one[A]
    (x * One.1[Multiplicative[A]]).val = x.val + One.1[Multiplicative[A]].val
    (x * One.1[Multiplicative[A]]).val = x.val + A.0
    x.val + A.0 = x.val
    (x * One.1[Multiplicative[A]]).val = x.val
    multiplicative_ext(x * One.1[Multiplicative[A]], x)
}

/// Multiplicative one is a left identity.
theorem multiplicative_mul_one_left[A: AddGroup](x: Multiplicative[A]) {
    One.1[Multiplicative[A]] * x = x
} by {
    multiplicative_val_mul(One.1[Multiplicative[A]], x)
    multiplicative_val_one[A]
    (One.1[Multiplicative[A]] * x).val = One.1[Multiplicative[A]].val + x.val
    (One.1[Multiplicative[A]] * x).val = A.0 + x.val
    A.0 + x.val = x.val
    (One.1[Multiplicative[A]] * x).val = x.val
    multiplicative_ext(One.1[Multiplicative[A]] * x, x)
}

/// Multiplicative tags satisfy the right identity law in primitive form.
theorem multiplicative_monoid_right_law[A: AddGroup] {
    forall(x: Multiplicative[A]) {
        Mul.mul(x, One.1[Multiplicative[A]]) = x
    }
} by {
    forall(x: Multiplicative[A]) {
        multiplicative_mul_one_right(x)
    }
}

/// Multiplicative tags satisfy the left identity law in primitive form.
theorem multiplicative_monoid_left_law[A: AddGroup] {
    forall(x: Multiplicative[A]) {
        Mul.mul(One.1[Multiplicative[A]], x) = x
    }
} by {
    forall(x: Multiplicative[A]) {
        multiplicative_mul_one_left(x)
    }
}

/// Multiplicative tags over an additive group form a monoid.
instance Multiplicative[A: AddGroup]: Monoid

/// Multiplicative inversion is a right inverse.
theorem multiplicative_inverse_right[A: AddGroup](x: Multiplicative[A]) {
    x * x.inverse = One.1[Multiplicative[A]]
} by {
    multiplicative_val_mul(x, x.inverse)
    multiplicative_val_inverse(x)
    (x * x.inverse).val = x.val + x.inverse.val
    (x * x.inverse).val = x.val + -x.val
    x.val + -x.val = A.0
    (x * x.inverse).val = A.0
    multiplicative_val_one[A]
    One.1[Multiplicative[A]].val = A.0
    (x * x.inverse).val = One.1[Multiplicative[A]].val
    multiplicative_ext(x * x.inverse, One.1[Multiplicative[A]])
}

/// Multiplicative tags satisfy the group inverse law in primitive form.
theorem multiplicative_group_inverse_law[A: AddGroup] {
    forall(x: Multiplicative[A]) {
        Mul.mul(x, x.inverse) = One.1[Multiplicative[A]]
    }
} by {
    forall(x: Multiplicative[A]) {
        multiplicative_inverse_right(x)
    }
}

/// Multiplicative tags over an additive group form a group.
instance Multiplicative[A: AddGroup]: Group {
    let inverse = Multiplicative[A].inverse
}

/// Multiplication in a multiplicative tag is commutative when the original additive group is commutative.
theorem multiplicative_mul_comm[A: AddCommGroup](x: Multiplicative[A], y: Multiplicative[A]) {
    x * y = y * x
} by {
    multiplicative_val_mul(x, y)
    multiplicative_val_mul(y, x)
    (x * y).val = x.val + y.val
    (y * x).val = y.val + x.val
    x.val + y.val = y.val + x.val
    (x * y).val = (y * x).val
    multiplicative_ext(x * y, y * x)
}

/// Multiplicative tags satisfy the commutative-semigroup law in primitive form.
theorem multiplicative_comm_semigroup_instance_law[A: AddCommGroup](x: Multiplicative[A], y: Multiplicative[A]) {
    Mul.mul(x, y) = Mul.mul(y, x)
} by {
    multiplicative_mul_comm(x, y)
}

/// Multiplicative tags over an additive commutative group form a commutative semigroup.
instance Multiplicative[A: AddCommGroup]: CommSemigroup

/// Multiplicative tags over an additive commutative group form a commutative monoid.
instance Multiplicative[A: AddCommGroup]: CommMonoid

/// Multiplicative tags over an additive commutative group form a commutative group.
instance Multiplicative[A: AddCommGroup]: CommGroup

/// True if additive zero acts trivially.
define add_action_identity_constraint[A: AddGroup, X](act: (A, X) -> X) -> Bool {
    forall(x: X) {
        act(A.0, x) = x
    }
}

/// True if addition in the group is composition of actions.
define add_action_add_constraint[A: AddGroup, X](act: (A, X) -> X) -> Bool {
    forall(a: A, b: A, x: X) {
        act(a + b, x) = act(a, act(b, x))
    }
}

/// True if a binary operation is a left additive action on a type.
define is_add_action[A: AddGroup, X](act: (A, X) -> X) -> Bool {
    add_action_add_constraint(act) and add_action_identity_constraint(act)
}

/// A left additive action on a type.
structure AddAction[A: AddGroup, X] {
    /// The action map.
    act: (A, X) -> X
} constraint {
    is_add_action(act)
}

/// Additive zero acts trivially.
theorem add_action_zero[A: AddGroup, X](a: AddAction[A, X], x: X) {
    a.act(A.0, x) = x
} by {
    is_add_action(a.act)
    add_action_identity_constraint(a.act)
    add_action_identity_constraint(a.act) = forall(y: X) {
        a.act(A.0, y) = y
    }
}

/// Addition in the group is composition of additive actions.
theorem add_action_add[A: AddGroup, X](a: AddAction[A, X], r: A, s: A, x: X) {
    a.act(r + s, x) = a.act(r, a.act(s, x))
} by {
    is_add_action(a.act)
    is_add_action(a.act) = add_action_add_constraint(a.act) and add_action_identity_constraint(a.act)
    add_action_add_constraint(a.act)
    add_action_add_constraint(a.act) = forall(u: A, v: A, y: X) {
        a.act(u + v, y) = a.act(u, a.act(v, y))
    }
}

/// The additive action obtained by retagging the acting group of a multiplicative action.
define additive_act_from_mul[G: Group, X](a: MulAction[G, X], g: Additive[G], x: X) -> X {
    a.act(g.val, x)
}

/// The retagged additive action evaluates through the underlying multiplicative value.
theorem additive_act_from_mul_apply[G: Group, X](a: MulAction[G, X], g: Additive[G], x: X) {
    additive_act_from_mul(a, g, x) = a.act(g.val, x)
}

/// The retagged additive action agrees with the original action on wrapped elements.
theorem additive_act_from_mul_of_mul[G: Group, X](a: MulAction[G, X], g: G, x: X) {
    additive_act_from_mul(a, additive_of_mul(g), x) = a.act(g, x)
} by {
    additive_act_from_mul(a, additive_of_mul(g), x) = a.act(additive_of_mul(g).val, x)
    additive_of_mul(g).val = g
}

/// Retagging a multiplicative action gives an additive action.
theorem additive_act_from_mul_is_add_action[G: Group, X](a: MulAction[G, X]) {
    is_add_action(additive_act_from_mul(a))
} by {
    forall(g: Additive[G], h: Additive[G], x: X) {
        additive_act_from_mul(a, g + h, x) = a.act((g + h).val, x)
        additive_val_add(g, h)
        (g + h).val = g.val * h.val
        additive_act_from_mul(a, g + h, x) = a.act(g.val * h.val, x)
        mul_action_mul(a, g.val, h.val, x)
        a.act(g.val * h.val, x) = a.act(g.val, a.act(h.val, x))
        additive_act_from_mul(a, h, x) = a.act(h.val, x)
        additive_act_from_mul(a, g, additive_act_from_mul(a, h, x)) =
            a.act(g.val, additive_act_from_mul(a, h, x))
        additive_act_from_mul(a, g, additive_act_from_mul(a, h, x)) =
            a.act(g.val, a.act(h.val, x))
        additive_act_from_mul(a, g + h, x) =
            additive_act_from_mul(a, g, additive_act_from_mul(a, h, x))
    }
    add_action_add_constraint(additive_act_from_mul(a))

    forall(x: X) {
        additive_act_from_mul(a, Zero.0[Additive[G]], x) =
            a.act(Zero.0[Additive[G]].val, x)
        additive_val_zero[G]
        Zero.0[Additive[G]].val = G.1
        additive_act_from_mul(a, Zero.0[Additive[G]], x) = a.act(G.1, x)
        mul_action_one(a, x)
        a.act(G.1, x) = x
        additive_act_from_mul(a, Zero.0[Additive[G]], x) = x
    }
    add_action_identity_constraint(additive_act_from_mul(a))
}

/// Bundle the additive action obtained from a multiplicative action by retagging.
let additive_action_from_mul[G: Group, X](a: MulAction[G, X]) -> result: AddAction[Additive[G], X] satisfy {
    AddAction[Additive[G], X].new(additive_act_from_mul(a)) = Option.some(result)
} by {
    additive_act_from_mul_is_add_action(a)
}

/// The bundled retagged additive action has the expected action map.
theorem additive_action_from_mul_act[G: Group, X](a: MulAction[G, X]) {
    additive_action_from_mul(a).act = additive_act_from_mul(a)
} by {
    AddAction[Additive[G], X].new(additive_act_from_mul(a)) = Option.some(additive_action_from_mul(a))
}

/// The bundled retagged additive action evaluates through the original multiplicative action.
theorem additive_action_from_mul_apply[G: Group, X](a: MulAction[G, X], g: Additive[G], x: X) {
    additive_action_from_mul(a).act(g, x) = a.act(g.val, x)
} by {
    additive_action_from_mul_act(a)
    additive_act_from_mul_apply(a, g, x)
}

/// The multiplicative action obtained by retagging the acting group of an additive action.
define multiplicative_act_from_add[A: AddGroup, X](a: AddAction[A, X], g: Multiplicative[A], x: X) -> X {
    a.act(g.val, x)
}

/// The retagged multiplicative action evaluates through the underlying additive value.
theorem multiplicative_act_from_add_apply[A: AddGroup, X](a: AddAction[A, X], g: Multiplicative[A], x: X) {
    multiplicative_act_from_add(a, g, x) = a.act(g.val, x)
}

/// The retagged multiplicative action agrees with the original action on wrapped elements.
theorem multiplicative_act_from_add_of_add[A: AddGroup, X](a: AddAction[A, X], r: A, x: X) {
    multiplicative_act_from_add(a, multiplicative_of_add(r), x) = a.act(r, x)
} by {
    multiplicative_act_from_add(a, multiplicative_of_add(r), x) = a.act(multiplicative_of_add(r).val, x)
    multiplicative_of_add(r).val = r
}

/// Retagging an additive action gives a multiplicative action.
theorem multiplicative_act_from_add_is_mul_action[A: AddGroup, X](a: AddAction[A, X]) {
    is_mul_action(multiplicative_act_from_add(a))
} by {
    forall(g: Multiplicative[A], h: Multiplicative[A], x: X) {
        multiplicative_act_from_add(a, g * h, x) = a.act((g * h).val, x)
        multiplicative_val_mul(g, h)
        (g * h).val = g.val + h.val
        multiplicative_act_from_add(a, g * h, x) = a.act(g.val + h.val, x)
        add_action_add(a, g.val, h.val, x)
        a.act(g.val + h.val, x) = a.act(g.val, a.act(h.val, x))
        multiplicative_act_from_add(a, h, x) = a.act(h.val, x)
        multiplicative_act_from_add(a, g, multiplicative_act_from_add(a, h, x)) =
            a.act(g.val, multiplicative_act_from_add(a, h, x))
        multiplicative_act_from_add(a, g, multiplicative_act_from_add(a, h, x)) =
            a.act(g.val, a.act(h.val, x))
        multiplicative_act_from_add(a, g * h, x) =
            multiplicative_act_from_add(a, g, multiplicative_act_from_add(a, h, x))
    }
    action_mul_constraint(multiplicative_act_from_add(a))

    forall(x: X) {
        multiplicative_act_from_add(a, One.1[Multiplicative[A]], x) =
            a.act(One.1[Multiplicative[A]].val, x)
        multiplicative_val_one[A]
        One.1[Multiplicative[A]].val = A.0
        multiplicative_act_from_add(a, One.1[Multiplicative[A]], x) = a.act(A.0, x)
        add_action_zero(a, x)
        a.act(A.0, x) = x
        multiplicative_act_from_add(a, One.1[Multiplicative[A]], x) = x
    }
    action_identity_constraint(multiplicative_act_from_add(a))
}

/// Bundle the multiplicative action obtained from an additive action by retagging.
let multiplicative_action_from_add[A: AddGroup, X](a: AddAction[A, X]) -> result: MulAction[Multiplicative[A], X] satisfy {
    MulAction[Multiplicative[A], X].new(multiplicative_act_from_add(a)) = Option.some(result)
} by {
    multiplicative_act_from_add_is_mul_action(a)
}

/// The bundled retagged multiplicative action has the expected action map.
theorem multiplicative_action_from_add_act[A: AddGroup, X](a: AddAction[A, X]) {
    multiplicative_action_from_add(a).act = multiplicative_act_from_add(a)
} by {
    MulAction[Multiplicative[A], X].new(multiplicative_act_from_add(a)) = Option.some(multiplicative_action_from_add(a))
}

/// The bundled retagged multiplicative action evaluates through the original additive action.
theorem multiplicative_action_from_add_apply[A: AddGroup, X](a: AddAction[A, X], g: Multiplicative[A], x: X) {
    multiplicative_action_from_add(a).act(g, x) = a.act(g.val, x)
} by {
    multiplicative_action_from_add_act(a)
    multiplicative_act_from_add_apply(a, g, x)
}
