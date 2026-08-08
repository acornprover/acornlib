/// Raw kernel-containment predicates for subgroup quotient lifts.

from algebra.group import Group
from algebra.subgroup import Subgroup

/// A representative map sends every element of the subgroup to identity.
define subgroup_maps_to_one_raw[G: Group, H: Group](phi: G -> H, n: Subgroup[G]) -> Bool {
    forall(a: G) {
        n.contains(a) implies phi(a) = H.1
    }
}

/// Raw kernel-containment applied at a particular subgroup element.
theorem subgroup_maps_to_one_raw_at[G: Group, H: Group](phi: G -> H, n: Subgroup[G], a: G) {
    subgroup_maps_to_one_raw(phi, n) and n.contains(a) implies phi(a) = H.1
} by {
    if subgroup_maps_to_one_raw(phi, n) and n.contains(a) {
        subgroup_maps_to_one_raw(phi, n) = forall(b: G) {
            n.contains(b) implies phi(b) = H.1
        }
        phi(a) = H.1
    }
}
