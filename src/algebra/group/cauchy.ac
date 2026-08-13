/// Cauchy's theorem: if a prime `p` divides the order of a finite group, the
/// group has an element of order `p`.
///
/// The proof is the classical tuple argument.  The set `X` of `p`-tuples of
/// group elements whose product is the identity has cardinality `|G|^(p-1)`,
/// which is divisible by `p` because `p` divides `|G|`.  Cyclic rotation by
/// one position acts on `X`; since `p` is prime, every orbit of this action
/// has size one or `p`, so the number of fixed points is congruent to the
/// number of tuples modulo `p`.  The fixed points are exactly the constant
/// tuples `(x, ..., x)` with `x.pow(p) = e`, so the number of solutions of
/// `x.pow(p) = e` is divisible by `p`, which forces a nontrivial solution;
/// `prime_power_period_implies_order` upgrades it to an element of order `p`.
///
/// This file develops the tuple, rotation, and counting machinery.  The
/// rotation orbit structure of lists is developed first (rotation applied
/// `length` times is the identity, and a rotation-fixed list is constant),
/// then the enumeration of all tuples of a fixed length over a finite list,
/// then the orbit partition of the tuple space and the fixed-point
/// congruence, and finally the theorem itself.

from algebra.group import Group, inverse_left, inverse_inverse, right_cancel, left_cancel
from algebra.group.group_action import list_prod, list_prod_nil, list_prod_cons, list_prod_const,
    const_list, rotate, rotate_cons, rotate_length, prime_power_period_implies_order,
    prime_divisor_one_or_self
from finite_group import FiniteGroup
from nat import Nat, mod_lt, div_mod_decomp, mod_of_decomp, mod_lte,
    pos_of_ne_zero, add_imp_sub, add_imp_sub_left, add_sub, add_one_right, add_one_left, suc_sub_one,
    lt_suc_right, lt_suc, not_lt_zero, trichotomy, lte_imp_not_lt, lt_imp_lte_suc, lte_and_lt,
    lte_antisymm, mul_to_one, divides_mul, gcd_of_prime, pow_add, pow_one, pow_distrib_mul,
    mul_suc_right, mul_comm, mul_zero_right, lte_mul, div_sub_mod, sub_zero,
    has_min, is_min, is_min_apply, is_min_false_below, false_below, false_below_apply,
    add_to_zero, lt_add_left, sub_pos, lte_add_left, lt_imp_lt_suc, zero_or_suc, lt_trans, mul_to_zero
from list import List, map, sum, sum_map_of_pointwise, map_map, map_contains,
    map_contains_of_contains, map_length,
    contains_imp_unique_contains, unique_contains_imp_contains, unique_preserves_contains,
    filter_equivalent_to_and, filter_preserves_unique, list_extensionality,
    add_contains_left, add_contains_right, add_length, range_contains_of_lt, length_range,
    map_add, map_range, not_unique_implies_duplicate, duplicate_implies_duplicate_idx
from data.basic.functions import is_injective_fn
from list import unique_implies_tail_unique, unique_length
from data.basic.set import Set, set_ext, list_set, list_set_contains_eq, set_eq_universal_of_forall_contains,
    unique_list_set_cardinality_is_length, cardinality_is_well_defined

numerals Nat

// ==== Iteration of a function ====

/// The `n`-fold iterate of a function: `iterate(f, 0, x) = x` and
/// `iterate(f, n + 1, x) = f(iterate(f, n, x))`.
define iterate[T](f: T -> T, n: Nat, x: T) -> T {
    match n {
        Nat.zero {
            x
        }
        Nat.suc(m) {
            f(iterate(f, m, x))
        }
    }
}

/// Iterating zero times is the identity.
theorem iterate_zero[T](f: T -> T, x: T) {
    iterate(f, Nat.0, x) = x
} by {
}

/// Iterating once more applies the function on the outside.
theorem iterate_suc[T](f: T -> T, n: Nat, x: T) {
    iterate(f, n.suc, x) = f(iterate(f, n, x))
} by {
}

/// Iterating once applies the function.
theorem iterate_one[T](f: T -> T, x: T) {
    iterate(f, Nat.1, x) = f(x)
} by {
    iterate(f, Nat.1, x) = iterate(f, Nat.0.suc, x)
    iterate(f, Nat.0.suc, x) = f(iterate(f, Nat.0, x))
    iterate(f, Nat.0, x) = x
}

/// Iteration splits over addition: the total iterate is the second iterate of the first.
theorem iterate_add[T](f: T -> T, a: Nat, b: Nat, x: T) {
    iterate(f, a + b, x) = iterate(f, b, iterate(f, a, x))
} by {
    define p(k: Nat) -> Bool {
        iterate(f, a + k, x) = iterate(f, k, iterate(f, a, x))
    }
    iterate(f, a + Nat.0, x) = iterate(f, a, x)
    iterate(f, Nat.0, iterate(f, a, x)) = iterate(f, a, x)
    p(Nat.0)
    forall(k: Nat) {
        if p(k) {
            iterate(f, a + k, x) = iterate(f, k, iterate(f, a, x))
            iterate(f, (a + k).suc, x) = f(iterate(f, a + k, x))
            iterate(f, a + k.suc, x) = iterate(f, (a + k).suc, x)
            iterate(f, k.suc, iterate(f, a, x)) = f(iterate(f, k, iterate(f, a, x)))
            iterate(f, a + k.suc, x) = iterate(f, k.suc, iterate(f, a, x))
            p(k.suc)
        }
    }
    forall(k: Nat) {
        p(k) implies p(k.suc)
    }
    p(Nat.0) and forall(k: Nat) {
        p(k) implies p(k.suc)
    }
    Nat.induction(p)
    p(b)
    iterate(f, a + b, x) = iterate(f, b, iterate(f, a, x))
}

/// Iteration commutes with a single application of the function.
theorem iterate_comm[T](f: T -> T, n: Nat, x: T) {
    iterate(f, n, f(x)) = f(iterate(f, n, x))
} by {
    iterate_add(f, Nat.1, n, x)
    iterate(f, Nat.1 + n, x) = iterate(f, n, iterate(f, Nat.1, x))
    iterate(f, Nat.1, x) = f(x)
    iterate(f, Nat.1 + n, x) = iterate(f, n, f(x))
    iterate_add(f, n, Nat.1, x)
    iterate(f, n + Nat.1, x) = iterate(f, Nat.1, iterate(f, n, x))
    iterate(f, Nat.1, iterate(f, n, x)) = f(iterate(f, n, x))
    iterate(f, n + Nat.1, x) = f(iterate(f, n, x))
    Nat.1 + n = n + Nat.1
    iterate(f, n, f(x)) = f(iterate(f, n, x))
}

// ==== Modular arithmetic lemmas ====

/// A natural below the modulus is its own remainder.
theorem lt_imp_mod_self(a: Nat, m: Nat) {
    a < m implies a.mod(m) = a
} by {
    if a < m {
        mod_of_decomp(Nat.0, a, m)
        (Nat.0 * m + a).mod(m) = a
        Nat.0 * m = Nat.0
        a.mod(m) = a
    }
}

/// The modulus is its own remainder only at zero: `m.mod(m) = 0`.
theorem mod_self_zero(m: Nat) {
    m != Nat.0 implies m.mod(m) = Nat.0
} by {
    if m != Nat.0 {
        pos_of_ne_zero(m)
        Nat.0 < m
        mod_of_decomp(Nat.1, Nat.0, m)
        (Nat.1 * m + Nat.0).mod(m) = Nat.0
        Nat.1 * m = m
        m.mod(m) = Nat.0
    }
}

/// Adding the modulus to a smaller number leaves the remainder unchanged.
theorem mod_add_self(a: Nat, m: Nat) {
    m != Nat.0 and a < m implies (a + m).mod(m) = a
} by {
    if m != Nat.0 and a < m {
        mod_of_decomp(Nat.1, a, m)
        (Nat.1 * m + a).mod(m) = a
        Nat.1 * m = m
        (m + a).mod(m) = a
        m + a = a + m
        (a + m).mod(m) = a
    }
}

/// Adding a multiple of the modulus does not change the remainder.
theorem mul_add_mod(q: Nat, r: Nat, m: Nat) {
    m != Nat.0 implies (q * m + r).mod(m) = r.mod(m)
} by {
    if m != Nat.0 {
        div_mod_decomp(r, m)
        r.div(m) * m + r.mod(m) = r
        q * m + r = q * m + (r.div(m) * m + r.mod(m))
        q * m + (r.div(m) * m + r.mod(m)) = (q * m + r.div(m) * m) + r.mod(m)
        q * m + r.div(m) * m = (q + r.div(m)) * m
        (q * m + r).mod(m) = ((q + r.div(m)) * m + r.mod(m)).mod(m)
        mod_lt(r, m)
        r.mod(m) < m
        mod_of_decomp(q + r.div(m), r.mod(m), m)
        ((q + r.div(m)) * m + r.mod(m)).mod(m) = r.mod(m)
        (q * m + r).mod(m) = r.mod(m)
    }
}

/// Remainders commute with addition: `(a mod m + k) mod m = (a + k) mod m`.
theorem mod_add_mod(a: Nat, k: Nat, m: Nat) {
    m != Nat.0 implies (a.mod(m) + k).mod(m) = (a + k).mod(m)
} by {
    if m != Nat.0 {
        div_mod_decomp(a, m)
        a.div(m) * m + a.mod(m) = a
        a + k = a.div(m) * m + (a.mod(m) + k)
        mul_add_mod(a.div(m), a.mod(m) + k, m)
        (a.div(m) * m + (a.mod(m) + k)).mod(m) = (a.mod(m) + k).mod(m)
        (a + k).mod(m) = (a.mod(m) + k).mod(m)
        (a.mod(m) + k).mod(m) = (a + k).mod(m)
    }
}

// ==== Rotation of lists ====

/// The `n`-fold rotation of a list.
define rotate_iter[G](t: List[G], n: Nat) -> List[G] {
    iterate(rotate[G], n, t)
}

/// Rotating zero times is the identity on lists.
theorem rotate_iter_zero[G](t: List[G]) {
    rotate_iter(t, Nat.0) = t
} by {
}

/// Rotating once more rotates the already-rotated list.
theorem rotate_iter_suc[G](t: List[G], n: Nat) {
    rotate_iter(t, n.suc) = rotate(rotate_iter(t, n))
} by {
}

/// Rotating once is rotation.
theorem rotate_iter_one[G](t: List[G]) {
    rotate_iter(t, Nat.1) = rotate(t)
} by {
}

/// Rotation splits over addition of the rotation count.
theorem rotate_iter_add[G](t: List[G], a: Nat, b: Nat) {
    rotate_iter(t, a + b) = rotate_iter(rotate_iter(t, a), b)
} by {
}

/// Rotation preserves the length of a list.
theorem rotate_iter_length[G](t: List[G], n: Nat) {
    rotate_iter(t, n).length = t.length
} by {
    define p(k: Nat) -> Bool {
        rotate_iter(t, k).length = t.length
    }
    rotate_iter(t, Nat.0).length = t.length
    p(Nat.0)
    forall(k: Nat) {
        if p(k) {
            rotate_iter(t, k).length = t.length
            rotate_iter(t, k.suc) = rotate(rotate_iter(t, k))
            rotate_iter(t, k.suc).length = rotate(rotate_iter(t, k)).length
            rotate_length(rotate_iter(t, k))
            rotate(rotate_iter(t, k)).length = rotate_iter(t, k).length
            rotate_iter(t, k.suc).length = t.length
            p(k.suc)
        }
    }
    forall(k: Nat) {
        p(k) implies p(k.suc)
    }
    p(Nat.0) and forall(k: Nat) {
        p(k) implies p(k.suc)
    }
    Nat.induction(p)
    p(n)
    rotate_iter(t, n).length = t.length
}

// ==== Element access under rotation ====

/// The element at index zero of a cons list is the head.
theorem get_idx_cons_zero[G](h: G, t: List[G]) {
    List.cons(h, t).get_idx(Nat.0) = Option.some(h)
} by {
    if Nat.0 > Nat.0 {
        false
    }
    List.cons(h, t).get_idx(Nat.0) = Option.some(h)
}

/// The element at a successor index of a cons list is the element at the
/// predecessor index of the tail.
theorem get_idx_cons_suc[G](h: G, t: List[G], m: Nat) {
    List.cons(h, t).get_idx(m.suc) = t.get_idx(m)
} by {
    if m.suc > Nat.0 {
        List.cons(h, t).get_idx(m.suc) = t.get_idx(m.suc - Nat.1)
        suc_sub_one(m)
        m.suc - Nat.1 = m
        List.cons(h, t).get_idx(m.suc) = t.get_idx(m)
    }
    if not (m.suc > Nat.0) {
        pos_of_ne_zero(m.suc)
        Nat.0 < m.suc
        m.suc > Nat.0
        false
    }
}

/// The nil list has no elements at any index.
theorem get_idx_nil[G](i: Nat) {
    List.nil[G].get_idx(i) = Option.none[G]
} by {
    match i {
        Nat.zero {
        }
        Nat.suc(m) {
        }
    }
}

/// Appending an element does not change the element at an index below the list length.
theorem append_get_idx_left[G](xs: List[G], x: G, i: Nat) {
    i < xs.length implies xs.append(x).get_idx(i) = xs.get_idx(i)
} by {
    define p(ys: List[G]) -> Bool {
        forall(j: Nat) {
            j < ys.length implies ys.append(x).get_idx(j) = ys.get_idx(j)
        }
    }
    p(List.nil[G])
    forall(head: G, tail: List[G]) {
        if p(tail) {
            forall(j: Nat) {
                if j < List.cons(head, tail).length {
                    match j {
                        Nat.zero {
                            List.cons(head, tail).append(x) = List.cons(head, tail.append(x))
                            get_idx_cons_zero(head, tail.append(x))
                            List.cons(head, tail.append(x)).get_idx(Nat.0) = Option.some(head)
                            get_idx_cons_zero(head, tail)
                            List.cons(head, tail).get_idx(Nat.0) = Option.some(head)
                            List.cons(head, tail).append(x).get_idx(j) = List.cons(head, tail).get_idx(j)
                        }
                        Nat.suc(m) {
                            List.cons(head, tail).append(x) = List.cons(head, tail.append(x))
                            get_idx_cons_suc(head, tail.append(x), m)
                            List.cons(head, tail.append(x)).get_idx(m.suc) = tail.append(x).get_idx(m)
                            get_idx_cons_suc(head, tail, m)
                            List.cons(head, tail).get_idx(m.suc) = tail.get_idx(m)
                            List.cons(head, tail).length = tail.length.suc
                            m.suc < tail.length.suc
                            lt_suc_right(m.suc, tail.length)
                            m.suc <= tail.length
                            lt_suc(m)
                            m < m.suc
                            lte_and_lt(m, m.suc, tail.length)
                            m < tail.length
                            tail.append(x).get_idx(m) = tail.get_idx(m)
                            List.cons(head, tail).append(x).get_idx(j) = List.cons(head, tail).get_idx(j)
                        }
                    }
                }
            }
            p(List.cons(head, tail))
        }
    }
    forall(head: G, tail: List[G]) {
        p(tail) implies p(List.cons(head, tail))
    }
    p(List.nil[G]) and forall(head: G, tail: List[G]) {
        p(tail) implies p(List.cons(head, tail))
    }
    List.induction(p)
    forall(ys: List[G]) {
        p(ys)
    }
    p(xs)
    i < xs.length implies xs.append(x).get_idx(i) = xs.get_idx(i)
}

/// Appending an element places it at the index equal to the original length.
theorem append_get_idx_right[G](xs: List[G], x: G) {
    xs.append(x).get_idx(xs.length) = Option.some(x)
} by {
    define p(ys: List[G]) -> Bool {
        ys.append(x).get_idx(ys.length) = Option.some(x)
    }
    List.nil[G].append(x) = List.cons(x, List.nil[G])
    List.nil[G].length = Nat.0
    get_idx_cons_zero(x, List.nil[G])
    List.cons(x, List.nil[G]).get_idx(Nat.0) = Option.some(x)
    List.nil[G].append(x).get_idx(List.nil[G].length) = Option.some(x)
    p(List.nil[G])
    forall(head: G, tail: List[G]) {
        if p(tail) {
            List.cons(head, tail).append(x) = List.cons(head, tail.append(x))
            List.cons(head, tail).length = tail.length.suc
            get_idx_cons_suc(head, tail.append(x), tail.length)
            List.cons(head, tail.append(x)).get_idx(tail.length.suc) =
                tail.append(x).get_idx(tail.length)
            tail.append(x).get_idx(tail.length) = Option.some(x)
            List.cons(head, tail).append(x).get_idx(List.cons(head, tail).length) = Option.some(x)
            p(List.cons(head, tail))
        }
    }
    forall(head: G, tail: List[G]) {
        p(tail) implies p(List.cons(head, tail))
    }
    p(List.nil[G]) and forall(head: G, tail: List[G]) {
        p(tail) implies p(List.cons(head, tail))
    }
    List.induction(p)
    forall(ys: List[G]) {
        p(ys)
    }
    p(xs)
    xs.append(x).get_idx(xs.length) = Option.some(x)
}

/// Rotating a list moves the element at index `i` to index `(i + 1) mod p`.
theorem rotate_get_idx[G](t: List[G], p: Nat, i: Nat) {
    t.length = p and p != Nat.0 and i < p implies
        rotate(t).get_idx(i) = t.get_idx((i + Nat.1).mod(p))
} by {
    if t.length = p and p != Nat.0 and i < p {
        match t {
            List.nil {
                t.length = Nat.0
                false
            }
            List.cons(head, rest) {
                rotate(List.cons(head, rest)) = rest.append(head)
                p = rest.length.suc
                i < rest.length.suc
                if i < rest.length {
                    rest.append(head).get_idx(i) = rest.get_idx(i)
                    get_idx_cons_suc(head, rest, i)
                    List.cons(head, rest).get_idx(i.suc) = rest.get_idx(i)
                    lt_imp_lte_suc(i, rest.length)
                    i.suc <= rest.length
                    lt_suc(rest.length)
                    rest.length < rest.length.suc
                    lte_and_lt(i.suc, rest.length, rest.length.suc)
                    i.suc < p
                    add_one_right(i)
                    i + Nat.1 = i.suc
                    lt_imp_mod_self(i.suc, p)
                    (i.suc).mod(p) = i.suc
                    (i + Nat.1).mod(p) = i.suc
                    List.cons(head, rest).get_idx((i + Nat.1).mod(p)) = rest.get_idx(i)
                    rotate(t).get_idx(i) = t.get_idx((i + Nat.1).mod(p))
                }
                if i >= rest.length {
                    lt_suc_right(i, rest.length)
                    i <= rest.length
                    lte_imp_not_lt(rest.length, i)
                    not (i < rest.length)
                    trichotomy(i, rest.length)
                    i = rest.length
                    add_one_right(rest.length)
                    rest.length + Nat.1 = rest.length.suc
                    i + Nat.1 = rest.length.suc
                    p = rest.length.suc
                    i + Nat.1 = p
                    mod_self_zero(p)
                    p.mod(p) = Nat.0
                    (i + Nat.1).mod(p) = Nat.0
                    rest.append(head).get_idx(rest.length) = Option.some(head)
                    get_idx_cons_zero(head, rest)
                    List.cons(head, rest).get_idx(Nat.0) = Option.some(head)
                    List.cons(head, rest).get_idx((i + Nat.1).mod(p)) = Option.some(head)
                    rotate(t).get_idx(i) = t.get_idx((i + Nat.1).mod(p))
                }
                rotate(t).get_idx(i) = t.get_idx((i + Nat.1).mod(p))
            }
        }
    }
}

/// The element at a rotation-shifted index of an iterated rotation.
theorem rotate_iter_get_idx[G](t: List[G], p: Nat, k: Nat, i: Nat) {
    t.length = p and p != Nat.0 and i < p implies
        rotate_iter(t, k).get_idx(i) = t.get_idx((i + k).mod(p))
} by {
    if t.length = p and p != Nat.0 and i < p {
        define q(n: Nat) -> Bool {
            forall(j: Nat) {
                j < p implies rotate_iter(t, n).get_idx(j) = t.get_idx((j + n).mod(p))
            }
        }
        forall(j: Nat) {
            if j < p {
                j + Nat.0 = j
                lt_imp_mod_self(j, p)
                j.mod(p) = j
                (j + Nat.0).mod(p) = j.mod(p)
                rotate_iter(t, Nat.0).get_idx(j) = t.get_idx(j)
                t.get_idx((j + Nat.0).mod(p)) = t.get_idx(j)
                rotate_iter(t, Nat.0).get_idx(j) = t.get_idx((j + Nat.0).mod(p))
            }
        }
        q(Nat.0)
        forall(n: Nat) {
            if q(n) {
                forall(j: Nat) {
                    if j < p {
                        rotate_iter(t, n.suc) = rotate(rotate_iter(t, n))
                        rotate_iter(t, n.suc).get_idx(j) = rotate(rotate_iter(t, n)).get_idx(j)
                        rotate_iter_length(t, n)
                        rotate_iter(t, n).length = p
                        rotate_iter(t, n).length != Nat.0
                        mod_lt(j + Nat.1, p)
                        (j + Nat.1).mod(p) < p
                        rotate_get_idx(rotate_iter(t, n), p, j)
                        rotate(rotate_iter(t, n)).get_idx(j) =
                            rotate_iter(t, n).get_idx((j + Nat.1).mod(p))
                        q(n)
                        rotate_iter(t, n).get_idx((j + Nat.1).mod(p)) =
                            t.get_idx(((j + Nat.1).mod(p) + n).mod(p))
                        mod_add_mod(j + Nat.1, n, p)
                        ((j + Nat.1).mod(p) + n).mod(p) = (j + Nat.1 + n).mod(p)
                        rotate_iter(t, n.suc).get_idx(j) = t.get_idx((j + Nat.1 + n).mod(p))
                        add_one_right(n)
                        n + Nat.1 = n.suc
                        j + n.suc = j + (n + Nat.1)
                        j + (n + Nat.1) = j + n + Nat.1
                        j + n + Nat.1 = j + Nat.1 + n
                        t.get_idx((j + Nat.1 + n).mod(p)) = t.get_idx((j + n.suc).mod(p))
                        rotate_iter(t, n.suc).get_idx(j) = t.get_idx((j + n.suc).mod(p))
                    }
                }
                q(n.suc)
            }
        }
        forall(n: Nat) {
            q(n) implies q(n.suc)
        }
        q(Nat.0) and forall(n: Nat) {
            q(n) implies q(n.suc)
        }
        Nat.induction(q)
        q(k)
        i < p
        rotate_iter(t, k).get_idx(i) = t.get_idx((i + k).mod(p))
    }
}

/// Rotating a list of length `p` by `p` positions is the identity.
theorem rotate_p_id[G](t: List[G], p: Nat) {
    t.length = p and p != Nat.0 implies rotate_iter(t, p) = t
} by {
    if t.length = p and p != Nat.0 {
        rotate_iter_length(t, p)
        rotate_iter(t, p).length = t.length
        forall(i: Nat) {
            if i < t.length {
                i < p
                mod_add_self(i, p)
                (i + p).mod(p) = i
                rotate_iter_get_idx(t, p, p, i)
                rotate_iter(t, p).get_idx(i) = t.get_idx((i + p).mod(p))
                rotate_iter(t, p).get_idx(i) = t.get_idx(i)
            }
        }
        rotate_iter(t, p).length = t.length and forall(i: Nat) {
            i < rotate_iter(t, p).length implies rotate_iter(t, p).get_idx(i) = t.get_idx(i)
        }
        list_extensionality(rotate_iter(t, p), t)
        rotate_iter(t, p) = t
    }
}

// ==== Constant lists and rotation fixed points ====

/// Appending one more copy of `x` to a constant list of copies of `x`.
theorem const_list_append[G](x: G, m: Nat) {
    const_list(x, m).append(x) = const_list(x, m + Nat.1)
} by {
    define p(k: Nat) -> Bool {
        const_list(x, k).append(x) = const_list(x, k + Nat.1)
    }
    List.nil[G].append(x) = List.cons(x, List.nil[G])
    const_list(x, Nat.1) = List.cons(x, const_list(x, Nat.0))
    const_list(x, Nat.0) = List.nil[G]
    List.nil[G].append(x) = const_list(x, Nat.0 + Nat.1)
    p(Nat.0)
    forall(k: Nat) {
        if p(k) {
            const_list(x, k).append(x) = const_list(x, k + Nat.1)
            const_list(x, k.suc) = List.cons(x, const_list(x, k))
            List.cons(x, const_list(x, k)).append(x) = List.cons(x, const_list(x, k).append(x))
            List.cons(x, const_list(x, k).append(x)) = List.cons(x, const_list(x, k + Nat.1))
            const_list(x, k.suc).append(x) = List.cons(x, const_list(x, k + Nat.1))
            const_list(x, (k + Nat.1).suc) = List.cons(x, const_list(x, k + Nat.1))
            add_one_right(k.suc)
            k.suc + Nat.1 = (k + Nat.1).suc
            const_list(x, k.suc + Nat.1) = List.cons(x, const_list(x, k + Nat.1))
            const_list(x, k.suc).append(x) = const_list(x, k.suc + Nat.1)
            p(k.suc)
        }
    }
    forall(k: Nat) {
        p(k) implies p(k.suc)
    }
    p(Nat.0) and forall(k: Nat) {
        p(k) implies p(k.suc)
    }
    Nat.induction(p)
    p(m)
    const_list(x, m).append(x) = const_list(x, m + Nat.1)
}

/// A constant list is fixed by rotation.
theorem const_rotate_fixed[G](x: G, n: Nat) {
    n != Nat.0 implies rotate(const_list(x, n)) = const_list(x, n)
} by {
    if n != Nat.0 {
        match n {
            Nat.zero {
                false
            }
            Nat.suc(m) {
                const_list(x, m.suc) = List.cons(x, const_list(x, m))
                rotate(List.cons(x, const_list(x, m))) = const_list(x, m).append(x)
                const_list_append(x, m)
                const_list(x, m).append(x) = const_list(x, m + Nat.1)
                add_one_right(m)
                m + Nat.1 = m.suc
                rotate(const_list(x, n)) = const_list(x, n)
            }
        }
    }
}

/// The head of a list, with the identity as a default for the empty list.
define head_of[G: Group](t: List[G]) -> G {
    match t {
        List.nil {
            G.1
        }
        List.cons(h, rest) {
            h
        }
    }
}

/// A list fixed by rotation is a constant list.
theorem rotate_fixed_imp_const[G: Group](t: List[G]) {
    rotate(t) = t implies t = const_list(head_of(t), t.length)
} by {
    define p(ys: List[G]) -> Bool {
        rotate(ys) = ys implies ys = const_list(head_of(ys), ys.length)
    }
    rotate(List.nil[G]) = List.nil[G]
    head_of(List.nil[G]) = G.1
    const_list(G.1, Nat.0) = List.nil[G]
    List.nil[G].length = Nat.0
    p(List.nil[G])
    forall(head: G, rest: List[G]) {
        if p(rest) {
            if rotate(List.cons(head, rest)) = List.cons(head, rest) {
                rotate(List.cons(head, rest)) = rest.append(head)
                rest.append(head) = List.cons(head, rest)
                match rest {
                    List.nil {
                        List.cons(head, rest) = List.cons(head, List.nil[G])
                        List.cons(head, rest).length = Nat.1
                        head_of(List.cons(head, rest)) = head
                        const_list(head, Nat.1) = List.cons(head, const_list(head, Nat.0))
                        const_list(head, Nat.0) = List.nil[G]
                        List.cons(head, rest) =
                            const_list(head_of(List.cons(head, rest)), List.cons(head, rest).length)
                    }
                    List.cons(head2, rest2) {
                        List.cons(head2, rest2).append(head) = List.cons(head2, rest2.append(head))
                        List.cons(head2, rest2.append(head)) = List.cons(head, List.cons(head2, rest2))
                        head2 = head
                        rest2.append(head) = List.cons(head2, rest2)
                        rotate(List.cons(head2, rest2)) = rest2.append(head2)
                        rotate(rest) = rest
                        p(rest)
                        rest = const_list(head_of(rest), rest.length)
                        head_of(rest) = head2
                        rest = const_list(head, rest.length)
                        List.cons(head, rest) = List.cons(head, const_list(head, rest.length))
                        const_list(head, rest.length.suc) = List.cons(head, const_list(head, rest.length))
                        List.cons(head, rest).length = rest.length.suc
                        head_of(List.cons(head, rest)) = head
                        const_list(head_of(List.cons(head, rest)), List.cons(head, rest).length) =
                            List.cons(head, const_list(head, rest.length))
                        const_list(head_of(List.cons(head, rest)), List.cons(head, rest).length) =
                            List.cons(head, rest)
                    }
                }
            }
            p(List.cons(head, rest))
        }
    }
    forall(head: G, rest: List[G]) {
        p(rest) implies p(List.cons(head, rest))
    }
    p(List.nil[G]) and forall(head: G, rest: List[G]) {
        p(rest) implies p(List.cons(head, rest))
    }
    List.induction(p)
    forall(ys: List[G]) {
        p(ys)
    }
    p(t)
    rotate(t) = t implies t = const_list(head_of(t), t.length)
}

/// A list fixed by rotation is constant, with the constant value exhibited.
theorem rotate_fixed_imp_exists_const[G: Group](t: List[G]) {
    rotate(t) = t implies exists(x: G) {
        t = const_list(x, t.length)
    }
} by {
    if rotate(t) = t {
        rotate_fixed_imp_const(t)
        t = const_list(head_of(t), t.length)
        exists(x: G) {
            t = const_list(x, t.length)
        }
    }
}

// ==== Periods of rotation ====

/// A list fixed by `a` rotations and by `b` rotations is fixed by `a + b` rotations.
theorem period_add[G](t: List[G], a: Nat, b: Nat) {
    rotate_iter(t, a) = t and rotate_iter(t, b) = t implies rotate_iter(t, a + b) = t
} by {
    if rotate_iter(t, a) = t and rotate_iter(t, b) = t {
        rotate_iter_add(t, a, b)
        rotate_iter(t, a + b) = rotate_iter(rotate_iter(t, a), b)
        rotate_iter(rotate_iter(t, a), b) = rotate_iter(t, b)
        rotate_iter(t, a + b) = t
    }
}

/// A list fixed by `a` rotations is fixed by any multiple of `a` rotations.
theorem period_mul[G](t: List[G], a: Nat, q: Nat) {
    rotate_iter(t, a) = t implies rotate_iter(t, a * q) = t
} by {
    if rotate_iter(t, a) = t {
        define p(k: Nat) -> Bool {
            rotate_iter(t, a * k) = t
        }
        a * Nat.0 = Nat.0
        rotate_iter(t, a * Nat.0) = t
        p(Nat.0)
        forall(k: Nat) {
            if p(k) {
                rotate_iter(t, a * k) = t
                mul_suc_right(a, k)
                a * k.suc = a + a * k
                a + a * k = a * k + a
                period_add(t, a * k, a)
                rotate_iter(t, a * k + a) = t
                rotate_iter(t, a * k.suc) = t
                p(k.suc)
            }
        }
        forall(k: Nat) {
            p(k) implies p(k.suc)
        }
        p(Nat.0) and forall(k: Nat) {
            p(k) implies p(k.suc)
        }
        Nat.induction(p)
        p(q)
        rotate_iter(t, a * q) = t
    }
}

/// A list fixed by `a` rotations and by `b` rotations is fixed by `a - b` rotations.
theorem period_sub[G](t: List[G], a: Nat, b: Nat) {
    b <= a and rotate_iter(t, a) = t and rotate_iter(t, b) = t implies rotate_iter(t, a - b) = t
} by {
    if b <= a and rotate_iter(t, a) = t and rotate_iter(t, b) = t {
        add_sub(a, b)
        a - b + b = a
        rotate_iter(t, a - b) = rotate_iter(rotate_iter(t, b), a - b)
        rotate_iter(t, b + (a - b)) = rotate_iter(rotate_iter(t, b), a - b)
        b + (a - b) = (a - b) + b
        b + (a - b) = a
        rotate_iter(t, a - b) = rotate_iter(t, a)
        rotate_iter(t, a - b) = t
    }
}

/// The remainder of a period modulo a period is a period.
theorem period_imp_mod_period[G](t: List[G], a: Nat, m: Nat) {
    m != Nat.0 and rotate_iter(t, a) = t and rotate_iter(t, m) = t implies
        rotate_iter(t, a.mod(m)) = t
} by {
    if m != Nat.0 and rotate_iter(t, a) = t and rotate_iter(t, m) = t {
        div_mod_decomp(a, m)
        a.div(m) * m + a.mod(m) = a
        period_mul(t, m, a.div(m))
        rotate_iter(t, m * a.div(m)) = t
        mul_comm(m, a.div(m))
        m * a.div(m) = a.div(m) * m
        rotate_iter(t, a.div(m) * m) = t
        a.div(m) * m + a.mod(m) = a
        add_imp_sub_left(a.div(m) * m, a.mod(m), a)
        a - a.div(m) * m = a.mod(m)
        a.div(m) * m <= a
        period_sub(t, a, a.div(m) * m)
        rotate_iter(t, a - a.div(m) * m) = t
        rotate_iter(t, a.mod(m)) = t
    }
}

/// True when `d` is a positive rotation count fixing the list `t`.
define period_witness[G](t: List[G], d: Nat) -> Bool {
    d != Nat.0 and rotate_iter(t, d) = t
}

/// Unfolding of the period-witness predicate.
theorem period_witness_unfold[G](t: List[G], d: Nat) {
    period_witness(t, d) = (d != Nat.0 and rotate_iter(t, d) = t)
} by {
}

/// A minimal positive period is no larger than any positive period.
theorem minimal_period_le_any[G](t: List[G], m: Nat, a: Nat) {
    is_min(period_witness(t), m) and period_witness(t, a) implies m <= a
} by {
    if is_min(period_witness(t), m) and period_witness(t, a) {
        is_min_false_below(period_witness(t), m)
        false_below(period_witness(t), m)
        if a < m {
            false_below_apply(period_witness(t), m, a)
            not period_witness(t)(a)
            period_witness(t)(a)
            false
        }
        not a < m
        m <= a
    }
}

/// The minimal positive period divides every period.
theorem minimal_period_divides_any[G](t: List[G], a: Nat, m: Nat) {
    is_min(period_witness(t), m) and rotate_iter(t, a) = t implies m.divides(a)
} by {
    if is_min(period_witness(t), m) and rotate_iter(t, a) = t {
        is_min_apply(period_witness(t), m)
        period_witness(t, m)
        period_witness_unfold(t, m)
        period_witness(t, m) = (m != Nat.0 and rotate_iter(t, m) = t)
        m != Nat.0
        rotate_iter(t, m) = t
        period_imp_mod_period(t, a, m)
        rotate_iter(t, a.mod(m)) = t
        mod_lt(a, m)
        a.mod(m) < m
        if a.mod(m) != Nat.0 {
            period_witness_unfold(t, a.mod(m))
            period_witness(t, a.mod(m)) =
                (a.mod(m) != Nat.0 and rotate_iter(t, a.mod(m)) = t)
            period_witness(t, a.mod(m))
            minimal_period_le_any(t, m, a.mod(m))
            m <= a.mod(m)
            lte_and_lt(a.mod(m), m, a.mod(m))
            a.mod(m) < a.mod(m)
            false
        }
        a.mod(m) = Nat.0
        div_sub_mod(a, m)
        m.divides(a - a.mod(m))
        sub_zero(a)
        a - Nat.0 = a
        m.divides(a)
    }
}

/// A list of prime length fixed by a nontrivial rotation count below the length
/// is fixed by a single rotation.
theorem period_lt_prime_imp_fixed[G](t: List[G], p: Nat, k: Nat) {
    t.length = p and p.is_prime and k > Nat.0 and k < p and
    rotate_iter(t, p) = t and rotate_iter(t, k) = t
    implies rotate(t) = t
} by {
    if t.length = p and p.is_prime and k > Nat.0 and k < p and
        rotate_iter(t, p) = t and rotate_iter(t, k) = t {
        p != Nat.0
        period_witness_unfold(t, p)
        period_witness(t, p) = (p != Nat.0 and rotate_iter(t, p) = t)
        period_witness(t, p)
        has_min(period_witness(t), p)
        let m: Nat satisfy { is_min(period_witness(t), m) }
        minimal_period_divides_any(t, k, m)
        m.divides(k)
        minimal_period_divides_any(t, p, m)
        m.divides(p)
        let c: Nat satisfy { m * c = k }
        if c = Nat.0 {
            m * Nat.0 = k
            Nat.0 = k
            k > Nat.0
            false
        }
        c != Nat.0
        lte_mul(m, c)
        m <= m * c
        m <= k
        lte_and_lt(m, k, p)
        m < p
        prime_divisor_one_or_self(p, m)
        m = Nat.1 or m = p
        if m = p {
            m < p
            p < p
            false
        }
        m = Nat.1
        is_min_apply(period_witness(t), m)
        period_witness(t, m)
        period_witness(t, Nat.1)
        period_witness_unfold(t, Nat.1)
        period_witness(t, Nat.1) = (Nat.1 != Nat.0 and rotate_iter(t, Nat.1) = t)
        rotate_iter(t, Nat.1) = t
        rotate_iter_one(t)
        rotate(t) = t
    }
}

// ==== Orbits of rotation ====

/// Equal maps on every member of a list give equal mapped lists.
theorem map_pointwise_eq[T, U](items: List[T], f: T -> U, g: T -> U) {
    (forall(x: T) { items.contains(x) implies f(x) = g(x) }) implies map(items, f) = map(items, g)
} by {
    define p(xs: List[T]) -> Bool {
        (forall(x: T) { xs.contains(x) implies f(x) = g(x) }) implies map(xs, f) = map(xs, g)
    }
    p(List.nil[T])
    forall(head: T, tail: List[T]) {
        if p(tail) {
            if forall(x: T) { List.cons(head, tail).contains(x) implies f(x) = g(x) } {
                f(head) = g(head)
                forall(x: T) {
                    if tail.contains(x) {
                        List.cons(head, tail).contains(x)
                        f(x) = g(x)
                    }
                }
                forall(x: T) { tail.contains(x) implies f(x) = g(x) }
                p(tail)
                p(tail) = ((forall(x: T) { tail.contains(x) implies f(x) = g(x) }) implies map(tail, f) = map(tail, g))
                (forall(x: T) { tail.contains(x) implies f(x) = g(x) }) implies map(tail, f) = map(tail, g)
                map(tail, f) = map(tail, g)
                map(List.cons(head, tail), f) = List.cons(f(head), map(tail, f))
                map(List.cons(head, tail), g) = List.cons(g(head), map(tail, g))
                map(List.cons(head, tail), f) = map(List.cons(head, tail), g)
            }
            p(List.cons(head, tail))
        }
    }
    forall(head: T, tail: List[T]) {
        p(tail) implies p(List.cons(head, tail))
    }
    p(List.nil[T]) and forall(head: T, tail: List[T]) {
        p(tail) implies p(List.cons(head, tail))
    }
    List.induction(p)
    forall(xs: List[T]) {
        p(xs)
    }
    p(items)
    p(items) = ((forall(x: T) { items.contains(x) implies f(x) = g(x) }) implies map(items, f) = map(items, g))
    (forall(x: T) { items.contains(x) implies f(x) = g(x) }) implies map(items, f) = map(items, g)
}

/// The constant function on the naturals returning `x`.
define const_fn[G](x: G) -> Nat -> G {
    function(k: Nat) { x }
}

/// The constant function evaluates to its value.
theorem const_fn_apply[G](x: G, n: Nat) {
    const_fn(x)(n) = x
} by {
}

/// Mapping a constant function over a range gives a constant list.
theorem map_range_const[G](x: G, p: Nat) {
    map[Nat, G](p.range, const_fn(x)) = const_list(x, p)
} by {
    define q(n: Nat) -> Bool {
        map[Nat, G](n.range, const_fn(x)) = const_list(x, n)
    }
    Nat.0.range = List.nil[Nat]
    map[Nat, G](List.nil[Nat], const_fn(x)) = List.nil[G]
    map[Nat, G](Nat.0.range, const_fn(x)) = List.nil[G]
    const_list(x, Nat.0) = List.nil[G]
    q(Nat.0)
    forall(n: Nat) {
        if q(n) {
            map[Nat, G](n.range, const_fn(x)) = const_list(x, n)
            n.suc.range = n.range.append(n)
            n.range.append(n) = n.range + List.singleton(n)
            map_add(n.range, List.singleton(n), const_fn(x))
            map[Nat, G](n.range + List.singleton(n), const_fn(x)) =
                map[Nat, G](n.range, const_fn(x)) +
                map[Nat, G](List.singleton(n), const_fn(x))
            List.singleton(n) = List.cons(n, List.nil[Nat])
            map[Nat, G](List.cons(n, List.nil[Nat]), const_fn(x)) =
                List.cons(const_fn(x)(n), map[Nat, G](List.nil[Nat], const_fn(x)))
            const_fn_apply(x, n)
            map[Nat, G](List.nil[Nat], const_fn(x)) = List.nil[G]
            List.cons(x, List.nil[G]) = List.singleton[G](x)
            map[Nat, G](List.singleton(n), const_fn(x)) = List.singleton(x)
            map[Nat, G](n.range + List.singleton(n), const_fn(x)) =
                const_list(x, n) + List.singleton(x)
            const_list(x, n) + List.singleton(x) = const_list(x, n).append(x)
            const_list_append(x, n)
            const_list(x, n).append(x) = const_list(x, n + Nat.1)
            add_one_right(n)
            n + Nat.1 = n.suc
            map[Nat, G](n.suc.range, const_fn(x)) = const_list(x, n.suc)
            q(n.suc)
        }
    }
    forall(n: Nat) {
        q(n) implies q(n.suc)
    }
    q(Nat.0) and forall(n: Nat) {
        q(n) implies q(n.suc)
    }
    Nat.induction(q)
    q(p)
    map[Nat, G](p.range, const_fn(x)) = const_list(x, p)
}

/// Deduplicating a cons list whose head already occurs in the tail drops the head.
theorem unique_cons_of_contains[T](head: T, tail: List[T]) {
    tail.contains(head) implies List.cons(head, tail).unique = tail.unique
} by {
    if tail.contains(head) {
        List.cons(head, tail).unique = tail.unique
    }
}

/// The deduplicated constant list of length one more than any `m` is a singleton.
theorem const_list_unique_suc[G](x: G, m: Nat) {
    const_list(x, m.suc).unique = List.singleton(x)
} by {
    define q(n: Nat) -> Bool {
        const_list(x, n.suc).unique = List.singleton(x)
    }
    const_list(x, Nat.1) = List.cons(x, const_list(x, Nat.0))
    const_list(x, Nat.0) = List.nil[G]
    if List.nil[G].contains(x) {
        false
    }
    List.cons(x, List.nil[G]).unique = List.cons(x, List.nil[G].unique)
    List.nil[G].unique = List.nil[G]
    List.cons(x, List.nil[G]).unique = List.cons(x, List.nil[G])
    List.cons(x, List.nil[G]) = List.singleton[G](x)
    const_list(x, Nat.1).unique = List.singleton(x)
    q(Nat.0)
    forall(n: Nat) {
        if q(n) {
            const_list(x, n.suc).unique = List.singleton(x)
            const_list(x, n.suc.suc) = List.cons(x, const_list(x, n.suc))
            const_list(x, n.suc).contains(x)
            unique_cons_of_contains(x, const_list(x, n.suc))
            List.cons(x, const_list(x, n.suc)).unique = const_list(x, n.suc).unique
            const_list(x, n.suc.suc).unique = List.singleton(x)
            q(n.suc)
        }
    }
    forall(n: Nat) {
        q(n) implies q(n.suc)
    }
    q(Nat.0) and forall(n: Nat) {
        q(n) implies q(n.suc)
    }
    Nat.induction(q)
    q(m)
    const_list(x, m.suc).unique = List.singleton(x)
}

/// The deduplicated constant list of positive length is a singleton.
theorem const_list_unique_pos[G](x: G, p: Nat) {
    p != Nat.0 implies const_list(x, p).unique = List.singleton(x)
} by {
    if p != Nat.0 {
        zero_or_suc(p)
        let m: Nat satisfy { p = m.suc }
        const_list_unique_suc(x, m)
        const_list(x, m.suc).unique = List.singleton(x)
        const_list(x, p).unique = List.singleton(x)
    }
}

/// A rotation-fixed list is fixed by every iterated rotation.
theorem all_iterates_fixed[G](t: List[G], k: Nat) {
    rotate(t) = t implies rotate_iter(t, k) = t
} by {
    if rotate(t) = t {
        define p(n: Nat) -> Bool {
            rotate_iter(t, n) = t
        }
        rotate_iter(t, Nat.0) = t
        p(Nat.0)
        forall(n: Nat) {
            if p(n) {
                rotate_iter(t, n) = t
                rotate_iter(t, n.suc) = rotate(rotate_iter(t, n))
                rotate_iter(t, n.suc) = rotate(t)
                rotate_iter(t, n.suc) = t
                p(n.suc)
            }
        }
        forall(n: Nat) {
            p(n) implies p(n.suc)
        }
        p(Nat.0) and forall(n: Nat) {
            p(n) implies p(n.suc)
        }
        Nat.induction(p)
        p(k)
        rotate_iter(t, k) = t
    }
}

/// The distinct rotations of a list of length `p`.
define rotation_orbit_list[G](t: List[G], p: Nat) -> List[List[G]] {
    map[Nat, List[G]](p.range, function(k: Nat) { rotate_iter(t, k) }).unique
}

/// The set of distinct rotations of a list of length `p`.
define rotation_orbit_set[G](t: List[G], p: Nat) -> Set[List[G]] {
    list_set(rotation_orbit_list(t, p))
}

/// A rotation-fixed list of positive length has singleton orbit.
theorem rotation_orbit_list_eq_singleton[G](t: List[G], p: Nat) {
    t.length = p and p != Nat.0 and rotate(t) = t implies
        rotation_orbit_list(t, p) = List.singleton(t)
} by {
    if t.length = p and p != Nat.0 and rotate(t) = t {
        forall(k: Nat) {
            rotate_iter(t, k) = t
        }
        map_pointwise_eq(p.range, function(k: Nat) { rotate_iter(t, k) }, const_fn(t))
        map[Nat, List[G]](p.range, function(k: Nat) { rotate_iter(t, k) }) =
            map[Nat, List[G]](p.range, const_fn(t))
        map_range_const(t, p)
        map[Nat, List[G]](p.range, const_fn(t)) = const_list(t, p)
        map[Nat, List[G]](p.range, function(k: Nat) { rotate_iter(t, k) }) = const_list(t, p)
        const_list_unique_pos(t, p)
        const_list(t, p).unique = List.singleton(t)
        rotation_orbit_list(t, p) = List.singleton(t)
    }
}

/// The rotation map over a prime-length range is injective in list form: no
/// two iterated rotations of a non-fixed list coincide.
theorem range_map_unique[G](t: List[G], p: Nat) {
    t.length = p and p.is_prime and rotate_iter(t, p) = t and rotate(t) != t implies
        map[Nat, List[G]](p.range, function(k: Nat) { rotate_iter(t, k) }).is_unique
} by {
    if t.length = p and p.is_prime and rotate_iter(t, p) = t and rotate(t) != t {
        if map[Nat, List[G]](p.range, function(k: Nat) { rotate_iter(t, k) }).is_unique {
        } else {
            not_unique_implies_duplicate(map[Nat, List[G]](p.range, function(k: Nat) { rotate_iter(t, k) }))
            let item: List[G] satisfy {
                map[Nat, List[G]](p.range, function(k: Nat) { rotate_iter(t, k) }).count(item) > 1
            }
            duplicate_implies_duplicate_idx(map[Nat, List[G]](p.range, function(k: Nat) { rotate_iter(t, k) }), item)
            let (i: Nat, j: Nat) satisfy {
                i < j and j < map[Nat, List[G]](p.range, function(k: Nat) { rotate_iter(t, k) }).length and
                map[Nat, List[G]](p.range, function(k: Nat) { rotate_iter(t, k) }).get_idx(i) = Option.some(item) and
                map[Nat, List[G]](p.range, function(k: Nat) { rotate_iter(t, k) }).get_idx(j) = Option.some(item)
            }
            map_length(p.range, function(k: Nat) { rotate_iter(t, k) })
            map[Nat, List[G]](p.range, function(k: Nat) { rotate_iter(t, k) }).length = p.range.length
            length_range(p)
            p.range.length = p
            map[Nat, List[G]](p.range, function(k: Nat) { rotate_iter(t, k) }).length = p
            j < p
            i < j
            lt_trans(i, j, p)
            i < p
            map_range(p, i, function(k: Nat) { rotate_iter(t, k) })
            map[Nat, List[G]](p.range, function(k: Nat) { rotate_iter(t, k) }).get_idx(i) =
                Option.some(rotate_iter(t, i))
            map_range(p, j, function(k: Nat) { rotate_iter(t, k) })
            map[Nat, List[G]](p.range, function(k: Nat) { rotate_iter(t, k) }).get_idx(j) =
                Option.some(rotate_iter(t, j))
            rotate_iter(t, i) = item
            rotate_iter(t, j) = item
            rotate_iter(t, i) = rotate_iter(t, j)
            lt_imp_lte_suc(j, p)
            j.suc <= p
            j <= p
            add_sub(p, j)
            p - j + j = p
            rotate_iter_add(t, j, p - j)
            rotate_iter(t, j + (p - j)) = rotate_iter(rotate_iter(t, j), p - j)
            j + (p - j) = p
            rotate_iter(t, p) = rotate_iter(rotate_iter(t, j), p - j)
            rotate_iter(rotate_iter(t, i), p - j) = rotate_iter(rotate_iter(t, j), p - j)
            rotate_iter_add(t, i, p - j)
            rotate_iter(t, i + (p - j)) = rotate_iter(rotate_iter(t, i), p - j)
            i + (p - j) = p - j + i
            rotate_iter(t, p - j + i) = rotate_iter(rotate_iter(t, i), p - j)
            rotate_iter(t, p - j + i) = rotate_iter(t, p)
            rotate_iter(t, p - j + i) = t
            lt_add_left(p - j, i, j)
            p - j + i < p - j + j
            p - j + i < p
            sub_pos(p, j)
            p - j > Nat.0
            if p - j + i = Nat.0 {
                add_to_zero(p - j, i)
                p - j = Nat.0
                p - j > Nat.0
                Nat.0 < p - j
                Nat.0 < Nat.0
                false
            }
            p - j + i != Nat.0
            p - j + i > Nat.0
            period_lt_prime_imp_fixed(t, p, p - j + i)
            rotate(t) = t
            rotate(t) != t
            false
        }
        map[Nat, List[G]](p.range, function(k: Nat) { rotate_iter(t, k) }).is_unique
    }
}

/// The rotation orbit list of a non-fixed list has length `p`.
theorem rotation_orbit_list_length_eq_p[G](t: List[G], p: Nat) {
    t.length = p and p.is_prime and rotate_iter(t, p) = t and rotate(t) != t implies
        rotation_orbit_list(t, p).length = p
} by {
    if t.length = p and p.is_prime and rotate_iter(t, p) = t and rotate(t) != t {
        range_map_unique(t, p)
        map[Nat, List[G]](p.range, function(k: Nat) { rotate_iter(t, k) }).is_unique
        rotation_orbit_list(t, p).length =
            map[Nat, List[G]](p.range, function(k: Nat) { rotate_iter(t, k) }).length
        map_length(p.range, function(k: Nat) { rotate_iter(t, k) })
        map[Nat, List[G]](p.range, function(k: Nat) { rotate_iter(t, k) }).length = p.range.length
        length_range(p)
        p.range.length = p
        rotation_orbit_list(t, p).length = p
    }
}

/// The rotation orbit list of a length-`p` list with `p` prime has length one or `p`.
theorem rotation_orbit_list_length_1_or_p[G](t: List[G], p: Nat) {
    t.length = p and p.is_prime and rotate_iter(t, p) = t implies
        (rotation_orbit_list(t, p).length = Nat.1 or rotation_orbit_list(t, p).length = p)
} by {
    if t.length = p and p.is_prime and rotate_iter(t, p) = t {
        if rotate(t) = t {
            p.is_prime
            Nat.1 < p
            if p = Nat.0 {
                Nat.1 < Nat.0
                not_lt_zero(Nat.1)
                false
            }
            p != Nat.0
            rotation_orbit_list_eq_singleton(t, p)
            rotation_orbit_list(t, p) = List.singleton(t)
            List.singleton[List[G]](t) = List.cons(t, List.nil[List[G]])
            List.cons(t, List.nil[List[G]]).length = List.nil[List[G]].length.suc
            List.nil[List[G]].length = Nat.0
            List.singleton[List[G]](t).length = Nat.1
            rotation_orbit_list(t, p).length = Nat.1
        }
        if rotate(t) != t {
            rotation_orbit_list_length_eq_p(t, p)
            rotation_orbit_list(t, p).length = p
        }
        rotation_orbit_list(t, p).length = Nat.1 or rotation_orbit_list(t, p).length = p
    }
}

