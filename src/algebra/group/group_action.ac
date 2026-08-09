/// Group actions: orbits partition the space they act on, the orbit-stabilizer
/// theorem in cardinality form, and Cauchy's theorem.
///
/// The definition of a group action (the `MulAction` structure with its two
/// axioms `act(G.1, x) = x` and `act(g * h, x) = act(g, act(h, x))`) and the
/// basic orbit and stabilizer machinery live in `algebra.group_action`; the
/// orbit-stabilizer theorem for finite groups is in
/// `finite_group.orbit_stabilizer`.  This file extends that machinery with the
/// orbit partition, the cardinality form of the orbit-stabilizer theorem, and
/// Cauchy's theorem together with the order argument at its heart.

from algebra.group import Group
from algebra.group_action import MulAction, orbit, orbit_relation, orbit_contains_self,
    orbit_contains_transitive, orbit_eq_of_contains, orbit_relation_eq_contains,
    orbit_relation_is_reflexive, orbit_relation_is_symmetric
from data.basic.set import Set, set_ext, equivalence_class, cardinality_is_well_defined
from data.basic.relation_basic import is_equivalence, is_reflexive, is_symmetric, is_transitive
from finite_group import FiniteGroup, finite_stabilizer, finite_orbit_list,
    finite_orbit_cardinality_is_length, finite_stabilizer_order_mul_finite_orbit_list_length_eq_group_order,
    cyclic_subgroup_of, cyclic_subgroup_order_pos,
    no_positive_period_below_cyclic_subgroup_order, pow_cyclic_subgroup_order_eq_identity,
    pow_mod_order
from list import List, add_length
from nat import Nat, mod_lt, div_sub_mod, sub_zero, trichotomy, pos_of_ne_zero,
    lt_suc_right, not_lt_zero, mul_zero_left, mul_zero_right, mul_one_right, pow_one,
    add_one_right

numerals Nat

// ==== Orbits partition the space ====

/// Two orbits that share a point are the same orbit.
theorem orbit_eq_of_intersection[G: Group, X](a: MulAction[G, X], x: X, y: X, z: X) {
    orbit(a, x).contains(z) and orbit(a, y).contains(z) implies orbit(a, x) = orbit(a, y)
} by {
    if orbit(a, x).contains(z) and orbit(a, y).contains(z) {
        orbit_eq_of_contains(a, x, z)
        orbit_eq_of_contains(a, y, z)
        orbit(a, x) = orbit(a, z)
        orbit(a, y) = orbit(a, z)
        orbit(a, x) = orbit(a, y)
    }
}

/// Any two orbits of a group action are equal or disjoint: the orbits partition
/// the space on which the group acts.
theorem orbit_eq_or_disjoint[G: Group, X](a: MulAction[G, X], x: X, y: X) {
    orbit(a, x) = orbit(a, y) or orbit(a, x).is_disjoint(orbit(a, y))
} by {
    if orbit(a, x) = orbit(a, y) {
    } else {
        forall(z: X) {
            if orbit(a, x).contains(z) and orbit(a, y).contains(z) {
                orbit_eq_of_intersection(a, x, y, z)
                false
            }
            not (orbit(a, x).contains(z) and orbit(a, y).contains(z))
        }
        orbit(a, x).is_disjoint(orbit(a, y))
    }
}

/// The orbit relation is transitive.
theorem orbit_relation_is_transitive[G: Group, X](a: MulAction[G, X]) {
    is_transitive(orbit_relation(a))
} by {
    if not is_transitive(orbit_relation(a)) {
        let (x: X, y: X, z: X) satisfy {
            orbit_relation(a, x, y) and orbit_relation(a, y, z) and not orbit_relation(a, x, z)
        }
        orbit_relation_eq_contains(a, x, y)
        orbit_relation_eq_contains(a, y, z)
        orbit_relation_eq_contains(a, x, z)
        orbit(a, x).contains(y)
        orbit(a, y).contains(z)
        orbit_contains_transitive(a, x, y, z)
        orbit(a, x).contains(z)
        orbit_relation(a, x, z)
        false
    }
}

/// The orbit relation is an equivalence relation.
theorem orbit_relation_is_equivalence[G: Group, X](a: MulAction[G, X]) {
    is_equivalence(orbit_relation(a))
} by {
    orbit_relation_is_reflexive(a)
    orbit_relation_is_symmetric(a)
    orbit_relation_is_transitive(a)
    is_reflexive(orbit_relation(a))
    is_symmetric(orbit_relation(a))
    is_transitive(orbit_relation(a))
    is_equivalence(orbit_relation(a))
}

/// The orbit of a point is exactly its equivalence class under the orbit
/// relation.
theorem orbit_eq_equivalence_class[G: Group, X](a: MulAction[G, X], x: X) {
    equivalence_class(orbit_relation(a), x) = orbit(a, x)
} by {
    forall(y: X) {
        equivalence_class(orbit_relation(a), x).contains(y) = orbit_relation(a, x, y)
        orbit_relation(a, x, y) = orbit(a, x).contains(y)
        equivalence_class(orbit_relation(a), x).contains(y) = orbit(a, x).contains(y)
    }
    set_ext(equivalence_class(orbit_relation(a), x), orbit(a, x))
}

/// The orbits cover the space: every point lies in its own orbit.
theorem orbit_covers[G: Group, X](a: MulAction[G, X], x: X) {
    orbit(a, x).contains(x)
} by {
    orbit_contains_self(a, x)
}

// ==== Orbit-stabilizer theorem in cardinality form ====

/// Orbit-stabilizer theorem: for a finite group acting on `X`, the stabilizer
/// order times the orbit cardinality is the group order:
/// `|Stab(x)| * |Orb(x)| = |G|`.
theorem orbit_stabilizer_order_mul_cardinality[G: FiniteGroup, X](
    a: MulAction[G, X],
    x: X,
    n: Nat
) {
    orbit(a, x).cardinality_is(n) implies finite_stabilizer(a, x).order * n = G.order
} by {
    if orbit(a, x).cardinality_is(n) {
        finite_orbit_cardinality_is_length(a, x)
        orbit(a, x).cardinality_is(finite_orbit_list(a, x).length)
        cardinality_is_well_defined(orbit(a, x), n, finite_orbit_list(a, x).length)
        n = finite_orbit_list(a, x).length
        finite_stabilizer_order_mul_finite_orbit_list_length_eq_group_order(a, x)
        finite_stabilizer(a, x).order * finite_orbit_list(a, x).length = G.order
        finite_stabilizer(a, x).order * n = G.order
    }
}

// ==== Cauchy's theorem ====

// The classical proof of Cauchy's theorem considers the action of the cyclic
// group of order `p` on the set of `p`-tuples `(x_1, ..., x_p)` of group
// elements with `x_1 * ... * x_p = e`, by cyclic rotation: a rotation fixed
// point is a constant tuple `(x, ..., x)` with `x.pow(p) = e`, and since the
// rotation orbits have size one or `p`, the number of fixed points is
// congruent to the number of tuples modulo `p`, which forces a nontrivial
// element of order `p`.  The tuple product and rotation machinery below sets
// this up, and the order argument that closes the proof is proved in full.

/// The product of a list of group elements, in order.
define list_prod[G: Group](t: List[G]) -> G {
    match t {
        List.nil {
            G.1
        }
        List.cons(head, tail) {
            head * list_prod(tail)
        }
    }
}

/// The product of the empty list is the identity element.
theorem list_prod_nil[G: Group] {
    list_prod(List.nil[G]) = G.1
} by {
}

/// The product of a cons list is the head times the product of the tail.
theorem list_prod_cons[G: Group](head: G, tail: List[G]) {
    list_prod(List.cons(head, tail)) = head * list_prod(tail)
} by {
}

/// The list of `n` copies of an element.
define const_list[G](x: G, n: Nat) -> List[G] {
    match n {
        Nat.zero {
            List.nil[G]
        }
        Nat.suc(m) {
            List.cons(x, const_list(x, m))
        }
    }
}

/// The product of a constant list is the corresponding power of the element.
theorem list_prod_const[G: Group](x: G, n: Nat) {
    list_prod(const_list(x, n)) = x.pow(n)
} by {
    let p: Nat -> Bool = function(k: Nat) {
        list_prod(const_list(x, k)) = x.pow(k)
    }
    list_prod(const_list(x, Nat.0)) = list_prod(List.nil[G])
    list_prod(List.nil[G]) = G.1
    x.pow(Nat.0) = G.1
    p(Nat.0)
    forall(k: Nat) {
        if p(k) {
            list_prod(const_list(x, k)) = x.pow(k)
            list_prod(const_list(x, k.suc)) = list_prod(List.cons(x, const_list(x, k)))
            list_prod(List.cons(x, const_list(x, k))) = x * list_prod(const_list(x, k))
            x.pow(k.suc) = x * x.pow(k)
            p(k.suc)
        }
    }
    forall(k: Nat) {
        p(k) implies p(k.suc)
    }
    p(Nat.0) and forall(k: Nat) {
        p(k) implies p(k.suc)
    }
    Nat.induction(p)
    p(n)
    list_prod(const_list(x, n)) = x.pow(n)
}

/// Rotating a list moves the head to the end.
define rotate[G](t: List[G]) -> List[G] {
    match t {
        List.nil {
            List.nil[G]
        }
        List.cons(head, tail) {
            tail.append(head)
        }
    }
}

/// Rotation of the empty list is the empty list.
theorem rotate_nil[G] {
    rotate(List.nil[G]) = List.nil[G]
} by {
}

/// Rotation of a cons list moves the head to the end.
theorem rotate_cons[G](head: G, tail: List[G]) {
    rotate(List.cons(head, tail)) = tail.append(head)
} by {
}

/// Appending an element increases the length of a list by one.
theorem append_length[T](items: List[T], item: T) {
    items.append(item).length = items.length + 1
} by {
    List.singleton(item) = List.cons(item, List.nil[T])
    List.cons(item, List.nil[T]).length = List.nil[T].length.suc
    List.nil[T].length = Nat.0
    List.singleton(item).length = Nat.0.suc
    Nat.0.suc = Nat.1
    items.append(item) = items + List.singleton(item)
    add_length(items, List.singleton(item))
    items.length + List.singleton(item).length = (items + List.singleton(item)).length
    items.append(item).length = items.length + List.singleton(item).length
    items.append(item).length = items.length + 1
}

/// Rotation preserves the length of a list.
theorem rotate_length[G](t: List[G]) {
    rotate(t).length = t.length
} by {
    define p(l: List[G]) -> Bool {
        rotate(l).length = l.length
    }
    p(List.nil[G])
    forall(head: G, tail: List[G]) {
        if p(tail) {
            rotate(tail).length = tail.length
            rotate_cons(head, tail)
            rotate(List.cons(head, tail)) = tail.append(head)
            rotate(List.cons(head, tail)).length = tail.append(head).length
            append_length(tail, head)
            tail.append(head).length = tail.length + 1
            rotate(List.cons(head, tail)).length = tail.length + 1
            add_one_right(tail.length)
            tail.length + 1 = tail.length.suc
            rotate(List.cons(head, tail)).length = tail.length.suc
            List.cons(head, tail).length = tail.length.suc
            rotate(List.cons(head, tail)).length = List.cons(head, tail).length
            p(List.cons(head, tail))
        }
    }
    forall(head: G, tail: List[G]) {
        p(tail) implies p(List.cons(head, tail))
    }
    p(List.nil[G]) and forall(head: G, tail: List[G]) {
        p(tail) implies p(List.cons(head, tail))
    }
    List.induction(p)
    forall(l: List[G]) {
        p(l)
    }
    p(t)
    rotate(t).length = t.length
}

/// A natural that is neither zero nor one is greater than one.
theorem ne_zero_ne_one_imp_lt_one(a: Nat) {
    a != Nat.0 and a != Nat.1 implies Nat.1 < a
} by {
    if a != Nat.0 and a != Nat.1 {
        pos_of_ne_zero(a)
        Nat.0 < a
        trichotomy(Nat.1, a)
        if a < Nat.1 {
            lt_suc_right(a, Nat.0)
            a = Nat.0 or a < Nat.0
            if a = Nat.0 {
                false
            }
            if a < Nat.0 {
                not_lt_zero(a)
                false
            }
        }
        if a = Nat.1 {
            false
        }
        Nat.1 < a
    }
}

/// Only one and a prime itself divide a prime.
theorem prime_divisor_one_or_self(p: Nat, d: Nat) {
    p.is_prime and d.divides(p) implies d = Nat.1 or d = p
} by {
    if p.is_prime and d.divides(p) {
        let k: Nat satisfy { d * k = p }
        p.is_prime
        Nat.1 < p
        not p.is_composite
        if d = Nat.1 {
        }
        if d != Nat.1 {
            if k = Nat.1 {
                d * Nat.1 = p
                mul_one_right(d)
                d * Nat.1 = d
                d = p
            }
            if k != Nat.1 {
                if k = Nat.0 {
                    d * Nat.0 = p
                    mul_zero_right(d)
                    d * Nat.0 = Nat.0
                    p = Nat.0
                    Nat.1 < p
                    Nat.1 < Nat.0
                    not_lt_zero(Nat.1)
                    false
                }
                k != Nat.0
                if d = Nat.0 {
                    Nat.0 * k = p
                    mul_zero_left(k)
                    Nat.0 * k = Nat.0
                    p = Nat.0
                    Nat.1 < p
                    Nat.1 < Nat.0
                    not_lt_zero(Nat.1)
                    false
                }
                d != Nat.0
                ne_zero_ne_one_imp_lt_one(k)
                Nat.1 < k
                ne_zero_ne_one_imp_lt_one(d)
                Nat.1 < d
                exists(b: Nat, c: Nat) {
                    Nat.1 < b and Nat.1 < c and p = b * c
                }
                p.is_composite
                false
            }
            d = p
        }
        d = Nat.1 or d = p
    }
}

/// An element of prime period `p` that is not the identity has order `p`.
theorem prime_power_period_implies_order[G: FiniteGroup](g: G, p: Nat) {
    p.is_prime and g != G.1 and g.pow(p) = G.1 implies cyclic_subgroup_of(g).order = p
} by {
    if p.is_prime and g != G.1 and g.pow(p) = G.1 {
        let d: Nat = cyclic_subgroup_of(g).order
        let r: Nat = p.mod(d)
        cyclic_subgroup_order_pos(g)
        d > Nat.0
        pow_cyclic_subgroup_order_eq_identity(g)
        g.pow(d) = G.1
        pow_mod_order(g, d, p)
        g.pow(r) = g.pow(p)
        g.pow(r) = G.1
        mod_lt(p, d)
        r < d
        if r > Nat.0 {
            no_positive_period_below_cyclic_subgroup_order(g, r)
            g.pow(r) != G.1
            false
        }
        r = Nat.0
        p.mod(d) = Nat.0
        div_sub_mod(p, d)
        d.divides(p - p.mod(d))
        sub_zero(p)
        p - Nat.0 = p
        d.divides(p)
        if d = Nat.1 {
            g.pow(d) = G.1
            g.pow(Nat.1) = G.1
            pow_one(g)
            g = G.1
            false
        }
        d != Nat.1
        prime_divisor_one_or_self(p, d)
        d = Nat.1 or d = p
        d = p
        cyclic_subgroup_of(g).order = p
    }
}

// Cauchy's theorem: if a prime `p` divides the order of a finite group, then
// the group has an element of order `p`.
//
//     theorem cauchy_theorem[G: FiniteGroup](p: Nat) {
//         p.is_prime and p.divides(G.order) implies exists(g: G) {
//             g != G.1 and g.pow(p) = G.1
//         }
//     }
//
// The statement is true by the classical tuple argument sketched above: the
// set of `p`-tuples with product the identity has cardinality `|G|^(p-1)`,
// which is divisible by `p`; cyclic rotation by one position acts on it with
// orbits of size one or `p`, so the number of fixed points (the constant
// tuples `(x, ..., x)` with `x.pow(p) = e`) is congruent to the number of
// tuples modulo `p`; that forces a nontrivial fixed point, and
// `prime_power_period_implies_order` upgrades it to an element of order `p`.
// The counting step (the cardinality of the tuple set and the orbit-size
// congruence) is not yet formalized; the tuple and rotation machinery above
// and the order argument are proved.
