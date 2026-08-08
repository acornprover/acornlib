/// Multiplication by a fixed group element as an embedding.

from data.basic.functions import is_injective_fn
from algebra.group import Group, left_cancel, right_cancel

/// A bundled injective function.
structure Embedding[A, B] {
    /// The underlying function.
    to_fun: A -> B
} constraint {
    is_injective_fn(to_fun)
}

/// Left multiplication by a fixed group element is injective.
theorem mul_left_injective[G: Group](g: G) {
    is_injective_fn(function(h: G) { g * h })
} by {
    forall(x: G, y: G) {
        if function(h: G) { g * h }(x) = function(h: G) { g * h }(y) {
            g * x = g * y
            left_cancel(g, x, y)
            x = y
        }
    }
}

/// Right multiplication by a fixed group element is injective.
theorem mul_right_injective[G: Group](g: G) {
    is_injective_fn(function(h: G) { h * g })
} by {
    forall(x: G, y: G) {
        if function(h: G) { h * g }(x) = function(h: G) { h * g }(y) {
            x * g = y * g
            right_cancel(g, x, y)
            x = y
        }
    }
}

/// Left multiplication by `g` as a bundled embedding.
let mulLeftEmbedding[G: Group](g: G) -> result: Embedding[G, G] satisfy {
    Embedding[G, G].new(function(h: G) { g * h }) = Option.some(result)
} by {
    mul_left_injective(g)
}

/// Right multiplication by `g` as a bundled embedding.
let mulRightEmbedding[G: Group](g: G) -> result: Embedding[G, G] satisfy {
    Embedding[G, G].new(function(h: G) { h * g }) = Option.some(result)
} by {
    mul_right_injective(g)
}
