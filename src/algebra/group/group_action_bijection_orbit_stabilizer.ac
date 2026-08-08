/// Orbit, stabilizer, and fixed-point transport along action bijections.

from algebra.group import Group, inverse_left
from pair import Pair
from algebra.group_action import MulAction, action_bijection, action_bijection_apply,
    mul_action_one, mul_action_mul, mul_action_inv_apply_apply,
    orbit, orbit_contains_action, orbit_contains_action_of_contains, orbit_eq_of_contains,
    stabilizer, stabilizer_fixes, stabilizer_contains_of_fixes,
    orbit_action_eq_iff_stabilizer_left_coset_raw, fixed_points
from algebra.group.group_action_fixed_points import stabilizer_contains_eq_fixed_points_contains
from algebra.group.product_action import product_action, product_action_stabilizer_eq_intersection
from algebra.group.product_action_fixed_points import product_action_fixed_points_contains_eq_coordinates
from algebra.subgroup import subgroup_intersection_contains_eq
from data.basic.functions import Inhabited, compose_bijection, inverse_bijection
from algebra.group.group_action_bijection import compose_action_bijection, inverse_bijection_action_bijection

/// The bundled inverse action bijection has the same map as acting by the inverse group element.
theorem inverse_bijection_action_bijection_apply_bridge[G: Group, X: Inhabited](
    a: MulAction[G, X], g: G, x: X
) {
    inverse_bijection(action_bijection(a, g)).map(x) = a.act(g.inverse, x)
} by {
    inverse_bijection_action_bijection(a, g)
    action_bijection_apply(a, g.inverse, x)
}

/// Composing action bijections applies the corresponding product action pointwise.
theorem compose_action_bijection_apply[G: Group, X](a: MulAction[G, X], g: G, h: G, x: X) {
    compose_bijection(action_bijection(a, g), action_bijection(a, h)).map(x) = a.act(g * h, x)
} by {
    compose_action_bijection(a, g, h)
    action_bijection_apply(a, g * h, x)
}

/// The inverse bijection composed after an action bijection is pointwise the identity.
theorem inverse_action_bijection_compose_apply_left[G: Group, X: Inhabited](a: MulAction[G, X], g: G, x: X) {
    compose_bijection(inverse_bijection(action_bijection(a, g)), action_bijection(a, g)).map(x) = x
} by {
    inverse_bijection_action_bijection(a, g)
    compose_action_bijection_apply(a, g.inverse, g, x)
    inverse_left(g)
    mul_action_mul(a, g.inverse, g, x)
    mul_action_one(a, x)
}

/// The action bijection composed after its inverse bijection is pointwise the identity.
theorem inverse_action_bijection_compose_apply_right[G: Group, X: Inhabited](a: MulAction[G, X], g: G, x: X) {
    compose_bijection(action_bijection(a, g), inverse_bijection(action_bijection(a, g))).map(x) = x
} by {
    inverse_bijection_action_bijection(a, g)
    compose_action_bijection_apply(a, g, g.inverse, x)
    mul_action_one(a, x)
    a.act(g * g.inverse, x) = x
}

/// Applying the inverse bundled action bijection lands back in the original orbit.
theorem inverse_action_bijection_maps_orbit_member_to_same_orbit[G: Group, X: Inhabited](
    a: MulAction[G, X], g: G, x: X, y: X
) {
    orbit(a, x).contains(y) implies orbit(a, x).contains(inverse_bijection(action_bijection(a, g)).map(y))
} by {
    if orbit(a, x).contains(y) {
        inverse_bijection_action_bijection_apply_bridge(a, g, y)
        orbit_contains_action_of_contains(a, x, y, g.inverse)
        orbit(a, x).contains(inverse_bijection(action_bijection(a, g)).map(y))
    }
}

/// The action bijection of a group element sends a point into its orbit.
theorem action_bijection_orbit_contains_image[G: Group, X](a: MulAction[G, X], g: G, x: X) {
    orbit(a, x).contains(action_bijection(a, g).map(x))
} by {
    action_bijection_apply(a, g, x)
    orbit_contains_action(a, x, g)
}

/// Moving the base point by an action bijection does not change its orbit.
theorem action_bijection_orbit_eq_image_base[G: Group, X](a: MulAction[G, X], g: G, x: X) {
    orbit(a, x) = orbit(a, action_bijection(a, g).map(x))
} by {
    action_bijection_orbit_contains_image(a, g, x)
    orbit_eq_of_contains(a, x, action_bijection(a, g).map(x))
}

/// Applying an action bijection to an orbit member keeps it in the same orbit.
theorem action_bijection_maps_orbit_member_to_same_orbit[G: Group, X](
    a: MulAction[G, X], g: G, x: X, y: X
) {
    orbit(a, x).contains(y) implies orbit(a, x).contains(action_bijection(a, g).map(y))
} by {
    if orbit(a, x).contains(y) {
        action_bijection_apply(a, g, y)
        orbit_contains_action_of_contains(a, x, y, g)
        orbit(a, x).contains(action_bijection(a, g).map(y))
    }
}

/// Applying an action bijection transports orbit membership after moving the base point too.
theorem action_bijection_maps_orbit_member[G: Group, X](
    a: MulAction[G, X], g: G, x: X, y: X
) {
    orbit(a, x).contains(y) implies
        orbit(a, action_bijection(a, g).map(x)).contains(action_bijection(a, g).map(y))
} by {
    if orbit(a, x).contains(y) {
        action_bijection_maps_orbit_member_to_same_orbit(a, g, x, y)
        action_bijection_orbit_eq_image_base(a, g, x)
        orbit(a, action_bijection(a, g).map(x)).contains(action_bijection(a, g).map(y))
    }
}

/// Orbit transport by an action bijection reflects back by acting with the inverse group element.
theorem action_bijection_reflects_orbit_member[G: Group, X](
    a: MulAction[G, X], g: G, x: X, y: X
) {
    orbit(a, action_bijection(a, g).map(x)).contains(action_bijection(a, g).map(y)) implies
        orbit(a, x).contains(y)
} by {
    if orbit(a, action_bijection(a, g).map(x)).contains(action_bijection(a, g).map(y)) {
        action_bijection_orbit_eq_image_base(a, g, x)
        orbit_contains_action_of_contains(a, x, action_bijection(a, g).map(y), g.inverse)
        orbit(a, x).contains(a.act(g.inverse, action_bijection(a, g).map(y)))
        action_bijection_apply(a, g, y)
        mul_action_inv_apply_apply(a, g, y)
        orbit(a, x).contains(y)
    }
}

/// Action bijections preserve and reflect orbit membership when both points are moved.
theorem action_bijection_orbit_contains_eq[G: Group, X](
    a: MulAction[G, X], g: G, x: X, y: X
) {
    orbit(a, x).contains(y) =
        orbit(a, action_bijection(a, g).map(x)).contains(action_bijection(a, g).map(y))
} by {
    if orbit(a, x).contains(y) {
        action_bijection_maps_orbit_member(a, g, x, y)
        orbit(a, action_bijection(a, g).map(x)).contains(action_bijection(a, g).map(y))
    }
    if orbit(a, action_bijection(a, g).map(x)).contains(action_bijection(a, g).map(y)) {
        action_bijection_reflects_orbit_member(a, g, x, y)
        orbit(a, x).contains(y)
    }
}

/// Stabilizer membership after moving a point by a group element is conjugated stabilizer membership.
theorem stabilizer_action_image_contains_eq_conjugate[G: Group, X](
    a: MulAction[G, X], g: G, h: G, x: X
) {
    stabilizer(a, a.act(g, x)).contains(h) = stabilizer(a, x).contains(g.inverse * h * g)
} by {
    if stabilizer(a, a.act(g, x)).contains(h) {
        stabilizer_fixes(a, a.act(g, x), h)
        mul_action_mul(a, h, g, x)
        orbit_action_eq_iff_stabilizer_left_coset_raw(a, x, g, h * g)
        stabilizer(a, x).contains(g.inverse * h * g)
    }
    if stabilizer(a, x).contains(g.inverse * h * g) {
        orbit_action_eq_iff_stabilizer_left_coset_raw(a, x, g, h * g)
        a.act(g, x) = a.act(h * g, x)
        mul_action_mul(a, h, g, x)
        stabilizer_contains_of_fixes(a, a.act(g, x), h)
        stabilizer(a, a.act(g, x)).contains(h)
    }
}

/// Stabilizer membership after moving by an action bijection is conjugated stabilizer membership.
theorem action_bijection_stabilizer_image_contains_eq_conjugate[G: Group, X](
    a: MulAction[G, X], g: G, h: G, x: X
) {
    stabilizer(a, action_bijection(a, g).map(x)).contains(h) =
        stabilizer(a, x).contains(g.inverse * h * g)
} by {
    action_bijection_apply(a, g, x)
    stabilizer_action_image_contains_eq_conjugate(a, g, h, x)
}

/// Fixed-point membership after moving by an action bijection is conjugated fixed-point membership.
theorem action_bijection_fixed_points_image_contains_eq_conjugate[G: Group, X](
    a: MulAction[G, X], g: G, h: G, x: X
) {
    fixed_points(a, h).contains(action_bijection(a, g).map(x)) =
        fixed_points(a, g.inverse * h * g).contains(x)
} by {
    stabilizer_contains_eq_fixed_points_contains(a, action_bijection(a, g).map(x), h)
    action_bijection_stabilizer_image_contains_eq_conjugate(a, g, h, x)
    stabilizer_contains_eq_fixed_points_contains(a, x, g.inverse * h * g)
}

/// A conjugated fixed point maps to a fixed point after applying an action bijection.
theorem action_bijection_maps_conjugate_fixed_points[G: Group, X](
    a: MulAction[G, X], g: G, h: G, x: X
) {
    fixed_points(a, g.inverse * h * g).contains(x) implies
        fixed_points(a, h).contains(action_bijection(a, g).map(x))
} by {
    if fixed_points(a, g.inverse * h * g).contains(x) {
        action_bijection_fixed_points_image_contains_eq_conjugate(a, g, h, x)
        fixed_points(a, h).contains(action_bijection(a, g).map(x))
    }
}

/// Fixed points of the moved point reflect to fixed points for the conjugated group element.
theorem action_bijection_reflects_fixed_points_to_conjugate[G: Group, X](
    a: MulAction[G, X], g: G, h: G, x: X
) {
    fixed_points(a, h).contains(action_bijection(a, g).map(x)) implies
        fixed_points(a, g.inverse * h * g).contains(x)
} by {
    if fixed_points(a, h).contains(action_bijection(a, g).map(x)) {
        action_bijection_fixed_points_image_contains_eq_conjugate(a, g, h, x)
        fixed_points(a, g.inverse * h * g).contains(x)
    }
}

/// Product-action fixed points transported by an action bijection split into conjugated coordinate fixed points.
theorem product_action_bijection_fixed_points_image_contains_eq_coordinates_conjugate[G: Group, X, Y](
    a: MulAction[G, X], b: MulAction[G, Y], g: G, h: G, p: Pair[X, Y]
) {
    fixed_points(product_action(a, b), h).contains(action_bijection(product_action(a, b), g).map(p)) =
        (fixed_points(a, g.inverse * h * g).contains(p.first) and
            fixed_points(b, g.inverse * h * g).contains(p.second))
} by {
    action_bijection_fixed_points_image_contains_eq_conjugate(product_action(a, b), g, h, p)
    product_action_fixed_points_contains_eq_coordinates(a, b, g.inverse * h * g, p)
}

/// Product-action stabilizers transported by an action bijection split into conjugated coordinate stabilizers.
theorem product_action_bijection_stabilizer_image_contains_eq_coordinates_conjugate[G: Group, X, Y](
    a: MulAction[G, X], b: MulAction[G, Y], g: G, h: G, p: Pair[X, Y]
) {
    stabilizer(product_action(a, b), action_bijection(product_action(a, b), g).map(p)).contains(h) =
        (stabilizer(a, p.first).contains(g.inverse * h * g) and
            stabilizer(b, p.second).contains(g.inverse * h * g))
} by {
    action_bijection_stabilizer_image_contains_eq_conjugate(product_action(a, b), g, h, p)
    product_action_stabilizer_eq_intersection(a, b, p, g.inverse * h * g)
    subgroup_intersection_contains_eq(stabilizer(a, p.first), stabilizer(b, p.second), g.inverse * h * g)
}
