/// Universal-property-facing quotient-lift API for group homomorphisms
/// whose kernel contains an arbitrary normal subgroup.

from algebra.group import Group, GroupHom, group_hom_mul, group_hom_one, group_hom_inv
from algebra.subgroup import Subgroup
from algebra.group.normal_subgroup_kernel_containment import subgroup_maps_to_one_raw,
    subgroup_maps_to_one_raw_at
from algebra.group.group_hom_kernel_eq import group_hom_eq_of_mul_inverse_maps_one
from data.basic.equivalence import QuotientOver, quotient_over_mk, quotient_over_mk_fields,
    rel_of_quotient_over_mk_eq, quotient_over_lift_to, quotient_over_lift_to_at,
    quotient_over_lift_to_at_has_representative
from algebra.group.normal_subgroup import is_normal_subgroup, normal_subgroup_rel,
    normal_subgroup_quotient_relation, normal_subgroup_quotient_relation_rel,
    normal_subgroup_quotient_project, normal_subgroup_quotient_project_eq_mk,
    normal_subgroup_quotient_mul, normal_subgroup_quotient_project_mul,
    normal_subgroup_quotient_one, normal_subgroup_quotient_project_one,
    normal_subgroup_quotient_inverse, normal_subgroup_quotient_project_inverse

/// A subgroup maps into the kernel of a group homomorphism: every element of
/// the subgroup is sent to the codomain identity.
define normal_subgroup_maps_to_one[G: Group, H: Group](
    f: GroupHom[G, H], n: Subgroup[G]
) -> Bool {
    subgroup_maps_to_one_raw(f.hom, n)
}

/// Kernel-containment applied to a particular subgroup element.
theorem normal_subgroup_maps_to_one_at[G: Group, H: Group](
    f: GroupHom[G, H], n: Subgroup[G], a: G
) {
    normal_subgroup_maps_to_one(f, n) and n.contains(a) implies f.hom(a) = H.1
} by {
    if normal_subgroup_maps_to_one(f, n) and n.contains(a) {
        subgroup_maps_to_one_raw_at(f.hom, n, a)
        f.hom(a) = H.1
    }
}

/// If a subgroup maps to the identity, its quotient relation is invisible to
/// the homomorphism.
theorem normal_subgroup_maps_to_one_rel_invariant[G: Group, H: Group](
    f: GroupHom[G, H], n: Subgroup[G], a: G, b: G
) {
    normal_subgroup_maps_to_one(f, n) and normal_subgroup_rel(n, a, b) implies
    f.hom(a) = f.hom(b)
} by {
    if normal_subgroup_maps_to_one(f, n) and normal_subgroup_rel(n, a, b) {
        normal_subgroup_maps_to_one_at(f, n, a * b.inverse)
        f.hom(a * b.inverse) = H.1
        group_hom_eq_of_mul_inverse_maps_one(f, a, b)
        f.hom(a) = f.hom(b)
    }
}

/// The map out of the quotient induced by a homomorphism whose kernel contains
/// the subgroup.  The well-definedness hypothesis is supplied to the projection
/// and operation theorems below.
define normal_subgroup_quotient_lift[G: Group, H: Group](
    n: Subgroup[G], f: GroupHom[G, H]
) -> QuotientOver[G] -> H {
    quotient_over_lift_to(f.hom)
}

/// The quotient lift agrees with the original homomorphism on canonical
/// quotient projections.
theorem normal_subgroup_quotient_lift_project[G: Group, H: Group](
    f: GroupHom[G, H], n: Subgroup[G], a: G
) {
    normal_subgroup_maps_to_one(f, n) implies
    normal_subgroup_quotient_lift(n, f, normal_subgroup_quotient_project(n, a)) = f.hom(a)
} by {
    if normal_subgroup_maps_to_one(f, n) {
        let qrel = normal_subgroup_quotient_relation(n)
        let q = normal_subgroup_quotient_project(n, a)
        normal_subgroup_quotient_project_eq_mk(n, a)
        quotient_over_mk_fields(qrel, a)
        quotient_over_lift_to_at_has_representative(f.hom, q)
        let b: G satisfy {
            quotient_over_mk(q.qrel, b) = q and quotient_over_lift_to_at(f.hom, q) = f.hom(b)
        }
        rel_of_quotient_over_mk_eq(qrel, b, a)
        qrel.rel(b, a)
        normal_subgroup_quotient_relation_rel(n)
        normal_subgroup_maps_to_one_rel_invariant(f, n, b, a)
        f.hom(b) = f.hom(a)
        quotient_over_lift_to(f.hom, q) = quotient_over_lift_to_at(f.hom, q)
        normal_subgroup_quotient_lift(n, f, normal_subgroup_quotient_project(n, a)) = f.hom(a)
    }
}

/// The lift carries multiplication of projected representatives to the product
/// of their homomorphic images.
theorem normal_subgroup_quotient_lift_mul_project[G: Group, H: Group](
    f: GroupHom[G, H], n: Subgroup[G], a: G, b: G
) {
    is_normal_subgroup(n) and normal_subgroup_maps_to_one(f, n) implies
    normal_subgroup_quotient_lift(n, f,
        normal_subgroup_quotient_mul(n,
            normal_subgroup_quotient_project(n, a),
            normal_subgroup_quotient_project(n, b))) = f.hom(a) * f.hom(b)
} by {
    if is_normal_subgroup(n) and normal_subgroup_maps_to_one(f, n) {
        normal_subgroup_quotient_project_mul(n, a, b)
        normal_subgroup_quotient_lift_project(f, n, a * b)
        group_hom_mul(f, a, b)
        normal_subgroup_quotient_lift(n, f,
            normal_subgroup_quotient_mul(n,
                normal_subgroup_quotient_project(n, a),
                normal_subgroup_quotient_project(n, b))) = f.hom(a) * f.hom(b)
    }
}

/// Multiplication preservation stated purely in terms of the quotient lift on
/// projected quotient elements.
theorem normal_subgroup_quotient_lift_mul_project_hom[G: Group, H: Group](
    f: GroupHom[G, H], n: Subgroup[G], a: G, b: G
) {
    is_normal_subgroup(n) and normal_subgroup_maps_to_one(f, n) implies
    normal_subgroup_quotient_lift(n, f,
        normal_subgroup_quotient_mul(n,
            normal_subgroup_quotient_project(n, a),
            normal_subgroup_quotient_project(n, b))) =
        normal_subgroup_quotient_lift(n, f, normal_subgroup_quotient_project(n, a)) *
        normal_subgroup_quotient_lift(n, f, normal_subgroup_quotient_project(n, b))
} by {
    if is_normal_subgroup(n) and normal_subgroup_maps_to_one(f, n) {
        normal_subgroup_quotient_lift_mul_project(f, n, a, b)
        normal_subgroup_quotient_lift_project(f, n, a)
        normal_subgroup_quotient_lift_project(f, n, b)
        f.hom(a) * f.hom(b) =
            normal_subgroup_quotient_lift(n, f, normal_subgroup_quotient_project(n, a)) *
            normal_subgroup_quotient_lift(n, f, normal_subgroup_quotient_project(n, b))
        normal_subgroup_quotient_lift(n, f,
            normal_subgroup_quotient_mul(n,
                normal_subgroup_quotient_project(n, a),
                normal_subgroup_quotient_project(n, b))) =
            normal_subgroup_quotient_lift(n, f, normal_subgroup_quotient_project(n, a)) *
            normal_subgroup_quotient_lift(n, f, normal_subgroup_quotient_project(n, b))
    }
}

/// The quotient lift sends the quotient identity to the codomain identity.
theorem normal_subgroup_quotient_lift_one[G: Group, H: Group](
    f: GroupHom[G, H], n: Subgroup[G]
) {
    normal_subgroup_maps_to_one(f, n) implies
    normal_subgroup_quotient_lift(n, f, normal_subgroup_quotient_one(n)) = H.1
} by {
    if normal_subgroup_maps_to_one(f, n) {
        normal_subgroup_quotient_project_one(n)
        normal_subgroup_quotient_lift_project(f, n, G.1)
        group_hom_one(f)
        normal_subgroup_quotient_lift(n, f, normal_subgroup_quotient_one(n)) = H.1
    }
}

/// The quotient lift carries inversion of projected representatives to inverse
/// images in the codomain.
theorem normal_subgroup_quotient_lift_inverse_project[G: Group, H: Group](
    f: GroupHom[G, H], n: Subgroup[G], a: G
) {
    is_normal_subgroup(n) and normal_subgroup_maps_to_one(f, n) implies
    normal_subgroup_quotient_lift(n, f,
        normal_subgroup_quotient_inverse(n, normal_subgroup_quotient_project(n, a))) =
        f.hom(a).inverse
} by {
    if is_normal_subgroup(n) and normal_subgroup_maps_to_one(f, n) {
        normal_subgroup_quotient_project_inverse(n, a)
        normal_subgroup_quotient_lift_project(f, n, a.inverse)
        group_hom_inv(f, a)
        normal_subgroup_quotient_lift(n, f,
            normal_subgroup_quotient_inverse(n, normal_subgroup_quotient_project(n, a))) =
            f.hom(a).inverse
    }
}

/// Inversion preservation stated in terms of the quotient lift on projected
/// quotient elements.
theorem normal_subgroup_quotient_lift_inverse_project_hom[G: Group, H: Group](
    f: GroupHom[G, H], n: Subgroup[G], a: G
) {
    is_normal_subgroup(n) and normal_subgroup_maps_to_one(f, n) implies
    normal_subgroup_quotient_lift(n, f,
        normal_subgroup_quotient_inverse(n, normal_subgroup_quotient_project(n, a))) =
        normal_subgroup_quotient_lift(n, f, normal_subgroup_quotient_project(n, a)).inverse
} by {
    if is_normal_subgroup(n) and normal_subgroup_maps_to_one(f, n) {
        normal_subgroup_quotient_lift_inverse_project(f, n, a)
        normal_subgroup_quotient_lift_project(f, n, a)
        normal_subgroup_quotient_lift(n, f,
            normal_subgroup_quotient_inverse(n, normal_subgroup_quotient_project(n, a))) =
            normal_subgroup_quotient_lift(n, f, normal_subgroup_quotient_project(n, a)).inverse
    }
}
