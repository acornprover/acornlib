/// The second isomorphism theorem: if `K` is a normal subgroup of `G` and
/// `H` is any subgroup, then the product set `HK` is a subgroup of `G`, and
/// the projection to `G/K` restricted to `H` has kernel `H ∩ K`, so the
/// induced map `H/(H ∩ K) -> G/K` is injective with image exactly the set of
/// `K`-cosets represented in `H` (i.e. `HK/K`).  The dual statement, with the
/// normality hypothesis on `H` instead of `K`, gives `HK ≤ G` and the
/// injective map `K/(H ∩ K) -> G/H` with image `HK/H`.
///
/// The quotient `G/K` is not a bundled `Group` instance in this library (the
/// congruence relation is carried by the value, not the type), so the
/// isomorphism content is stated at the level of the quotient projections,
/// mirroring `first_iso_deep`.

from algebra.group import Group, inverse_inverse, inverse_mul
from algebra.subgroup import Subgroup, identity_constraint, closure_constraint,
    inverse_constraint, subgroup_constraint, subgroup_contains_identity,
    subgroup_mul_mem, subgroup_inv_mem, subgroup_intersection,
    subgroup_intersection_contains_eq, subgroup_intersection_subset_left,
    subgroup_intersection_subset_right
from algebra.group.normal_subgroup import is_normal_subgroup, normal_conjugate_mem,
    normal_subgroup_rel, normal_subgroup_quotient_project,
    normal_subgroup_quotient_project_eq_iff_rel, normal_subgroup_quotient_project_eq_of_rel,
    normal_subgroup_rel_of_quotient_project_eq, normal_subgroup_quotient_one,
    normal_subgroup_quotient_project_eq_one_iff_contains,
    normal_subgroup_quotient_project_mul, normal_subgroup_quotient_project_inverse,
    normal_subgroup_quotient_mul, normal_subgroup_quotient_mul_one,
    normal_subgroup_quotient_one_mul, normal_subgroup_quotient_inverse,
    normal_subgroup_quotient_project_eq_mk
from data.basic.equivalence import QuotientOver

// ==== The product of two subgroups ====

/// Membership in the product set `HK`: an element of the form `h * k` with
/// `h` in `H` and `k` in `K`.
define subgroup_product_contains[G: Group](h: Subgroup[G], k: Subgroup[G], x: G) -> Bool {
    exists(a: G, b: G) {
        h.contains(a) and k.contains(b) and x = a * b
    }
}

/// The product set `HK` contains the identity element.
theorem subgroup_product_identity_constraint[G: Group](h: Subgroup[G], k: Subgroup[G]) {
    identity_constraint(subgroup_product_contains(h, k))
} by {
    subgroup_contains_identity(h)
    subgroup_contains_identity(k)
    G.1 = G.1 * G.1
    subgroup_product_contains(h, k, G.1)
    identity_constraint(subgroup_product_contains(h, k))
}

/// If `K` is normal, an element of `K` can be moved past an element of `H`:
/// `b * a = a * kp` for some `kp` in `K`.
theorem subgroup_product_commute_left[G: Group](h: Subgroup[G], k: Subgroup[G], a: G, b: G) {
    is_normal_subgroup(k) and h.contains(a) and k.contains(b) implies
    exists(kp: G) {
        k.contains(kp) and b * a = a * kp
    }
} by {
    if is_normal_subgroup(k) and h.contains(a) and k.contains(b) {
        subgroup_inv_mem(h, a)
        normal_conjugate_mem(k, a.inverse, b)
        k.contains(a.inverse * b * a.inverse.inverse)
        inverse_inverse(a)
        k.contains(a.inverse * b * a)
        a * (a.inverse * b * a) = a * a.inverse * b * a
        a * a.inverse * b * a = G.1 * b * a
        G.1 * b * a = b * a
        a * (a.inverse * b * a) = b * a
        exists(kp: G) {
            k.contains(kp) and b * a = a * kp
        }
    }
}

/// The product set `HK` is closed under multiplication when `K` is normal.
theorem subgroup_product_closure_constraint[G: Group](h: Subgroup[G], k: Subgroup[G]) {
    is_normal_subgroup(k) implies closure_constraint(subgroup_product_contains(h, k))
} by {
    if is_normal_subgroup(k) {
        forall(x: G, y: G) {
            if subgroup_product_contains(h, k, x) and subgroup_product_contains(h, k, y) {
                let (a: G, b: G) satisfy {
                    h.contains(a) and k.contains(b) and x = a * b
                }
                let (c: G, d: G) satisfy {
                    h.contains(c) and k.contains(d) and y = c * d
                }
                subgroup_product_commute_left(h, k, c, b)
                let kp: G satisfy {
                    k.contains(kp) and b * c = c * kp
                }
                subgroup_mul_mem(h, a, c)
                subgroup_mul_mem(k, kp, d)
                x * y = (a * b) * (c * d)
                (a * b) * (c * d) = a * (b * c) * d
                b * c = c * kp
                a * (b * c) * d = a * (c * kp) * d
                a * (c * kp) * d = (a * c) * (kp * d)
                x * y = (a * c) * (kp * d)
                exists(a1: G, b1: G) {
                    h.contains(a1) and k.contains(b1) and x * y = a1 * b1
                }
                subgroup_product_contains(h, k, x * y)
            }
        }
        closure_constraint(subgroup_product_contains(h, k))
    }
}

/// The product set `HK` is closed under inverses when `K` is normal.
theorem subgroup_product_inverse_constraint[G: Group](h: Subgroup[G], k: Subgroup[G]) {
    is_normal_subgroup(k) implies inverse_constraint(subgroup_product_contains(h, k))
} by {
    if is_normal_subgroup(k) {
        forall(x: G) {
            if subgroup_product_contains(h, k, x) {
                let (a: G, b: G) satisfy {
                    h.contains(a) and k.contains(b) and x = a * b
                }
                subgroup_inv_mem(h, a)
                subgroup_inv_mem(k, b)
                subgroup_product_commute_left(h, k, a.inverse, b.inverse)
                let kp: G satisfy {
                    k.contains(kp) and b.inverse * a.inverse = a.inverse * kp
                }
                inverse_mul(a, b)
                (a * b).inverse = b.inverse * a.inverse
                x.inverse = b.inverse * a.inverse
                b.inverse * a.inverse = a.inverse * kp
                x.inverse = a.inverse * kp
                exists(a1: G, b1: G) {
                    h.contains(a1) and k.contains(b1) and x.inverse = a1 * b1
                }
                subgroup_product_contains(h, k, x.inverse)
            }
        }
        inverse_constraint(subgroup_product_contains(h, k))
    }
}

/// The product set `HK` is a subgroup of `G` when `K` is normal.
theorem subgroup_product_constraint[G: Group](h: Subgroup[G], k: Subgroup[G]) {
    is_normal_subgroup(k) implies subgroup_constraint(subgroup_product_contains(h, k))
} by {
    if is_normal_subgroup(k) {
        subgroup_product_identity_constraint(h, k)
        identity_constraint(subgroup_product_contains(h, k))
        subgroup_product_closure_constraint(h, k)
        closure_constraint(subgroup_product_contains(h, k))
        subgroup_product_inverse_constraint(h, k)
        inverse_constraint(subgroup_product_contains(h, k))
        subgroup_constraint(subgroup_product_contains(h, k))
    }
}

/// Every element of `H` lies in the product `HK`.
theorem subgroup_product_contains_of_left[G: Group](h: Subgroup[G], k: Subgroup[G], a: G) {
    h.contains(a) implies subgroup_product_contains(h, k, a)
} by {
    if h.contains(a) {
        subgroup_contains_identity(k)
        a = a * G.1
        subgroup_product_contains(h, k, a)
    }
}

/// Every element of `K` lies in the product `HK`.
theorem subgroup_product_contains_of_right[G: Group](h: Subgroup[G], k: Subgroup[G], b: G) {
    k.contains(b) implies subgroup_product_contains(h, k, b)
} by {
    if k.contains(b) {
        subgroup_contains_identity(h)
        b = G.1 * b
        subgroup_product_contains(h, k, b)
    }
}

// ==== The product when the left subgroup is normal ====

/// If `H` is normal, an element of `K` can be moved past an element of `H`:
/// `b * a = hp * b` for some `hp` in `H`.
theorem subgroup_product_commute_right[G: Group](h: Subgroup[G], k: Subgroup[G], a: G, b: G) {
    is_normal_subgroup(h) and h.contains(a) and k.contains(b) implies
    exists(hp: G) {
        h.contains(hp) and b * a = hp * b
    }
} by {
    if is_normal_subgroup(h) and h.contains(a) and k.contains(b) {
        normal_conjugate_mem(h, b, a)
        h.contains(b * a * b.inverse)
        b * a = (b * a * b.inverse) * b
        exists(hp: G) {
            h.contains(hp) and b * a = hp * b
        }
    }
}

/// The product set `HK` is closed under multiplication when `H` is normal.
theorem subgroup_product_closure_constraint_right[G: Group](h: Subgroup[G], k: Subgroup[G]) {
    is_normal_subgroup(h) implies closure_constraint(subgroup_product_contains(h, k))
} by {
    if is_normal_subgroup(h) {
        forall(x: G, y: G) {
            if subgroup_product_contains(h, k, x) and subgroup_product_contains(h, k, y) {
                let (a: G, b: G) satisfy {
                    h.contains(a) and k.contains(b) and x = a * b
                }
                let (c: G, d: G) satisfy {
                    h.contains(c) and k.contains(d) and y = c * d
                }
                subgroup_product_commute_right(h, k, c, b)
                let hp: G satisfy {
                    h.contains(hp) and b * c = hp * b
                }
                subgroup_mul_mem(h, a, hp)
                subgroup_mul_mem(k, b, d)
                x * y = (a * b) * (c * d)
                (a * b) * (c * d) = a * (b * c) * d
                b * c = hp * b
                a * (b * c) * d = a * (hp * b) * d
                a * (hp * b) * d = (a * hp) * (b * d)
                x * y = (a * hp) * (b * d)
                exists(a1: G, b1: G) {
                    h.contains(a1) and k.contains(b1) and x * y = a1 * b1
                }
                subgroup_product_contains(h, k, x * y)
            }
        }
        closure_constraint(subgroup_product_contains(h, k))
    }
}

/// The product set `HK` is closed under inverses when `H` is normal.
theorem subgroup_product_inverse_constraint_right[G: Group](h: Subgroup[G], k: Subgroup[G]) {
    is_normal_subgroup(h) implies inverse_constraint(subgroup_product_contains(h, k))
} by {
    if is_normal_subgroup(h) {
        forall(x: G) {
            if subgroup_product_contains(h, k, x) {
                let (a: G, b: G) satisfy {
                    h.contains(a) and k.contains(b) and x = a * b
                }
                subgroup_inv_mem(h, a)
                subgroup_inv_mem(k, b)
                subgroup_product_commute_right(h, k, a.inverse, b.inverse)
                let hp: G satisfy {
                    h.contains(hp) and b.inverse * a.inverse = hp * b.inverse
                }
                inverse_mul(a, b)
                (a * b).inverse = b.inverse * a.inverse
                x.inverse = b.inverse * a.inverse
                b.inverse * a.inverse = hp * b.inverse
                x.inverse = hp * b.inverse
                exists(a1: G, b1: G) {
                    h.contains(a1) and k.contains(b1) and x.inverse = a1 * b1
                }
                subgroup_product_contains(h, k, x.inverse)
            }
        }
        inverse_constraint(subgroup_product_contains(h, k))
    }
}

/// The product set `HK` is a subgroup of `G` when `H` is normal.
theorem subgroup_product_constraint_right[G: Group](h: Subgroup[G], k: Subgroup[G]) {
    is_normal_subgroup(h) implies subgroup_constraint(subgroup_product_contains(h, k))
} by {
    if is_normal_subgroup(h) {
        subgroup_product_identity_constraint(h, k)
        identity_constraint(subgroup_product_contains(h, k))
        subgroup_product_closure_constraint_right(h, k)
        closure_constraint(subgroup_product_contains(h, k))
        subgroup_product_inverse_constraint_right(h, k)
        inverse_constraint(subgroup_product_contains(h, k))
        subgroup_constraint(subgroup_product_contains(h, k))
    }
}

// ==== Normality of the intersection within the left subgroup ====

/// The intersection `H ∩ K` is closed under conjugation by elements of `H`
/// when `K` is normal: `H ∩ K` is normal inside `H`.
theorem subgroup_intersection_normal_in_left[G: Group](h: Subgroup[G], k: Subgroup[G], g: G, x: G) {
    is_normal_subgroup(k) and h.contains(g) and h.intersection(k).contains(x) implies
    h.intersection(k).contains(g * x * g.inverse)
} by {
    if is_normal_subgroup(k) and h.contains(g) and h.intersection(k).contains(x) {
        subgroup_intersection_subset_left(h, k, x)
        subgroup_intersection_subset_right(h, k, x)
        subgroup_inv_mem(h, g)
        subgroup_mul_mem(h, g, x)
        subgroup_mul_mem(h, g * x, g.inverse)
        h.contains(g * x * g.inverse)
        normal_conjugate_mem(k, g, x)
        k.contains(g * x * g.inverse)
        subgroup_intersection_contains_eq(h, k, g * x * g.inverse)
        h.intersection(k).contains(g * x * g.inverse)
    }
}

/// The intersection `H ∩ K` is closed under conjugation by elements of `K`
/// when `H` is normal: `H ∩ K` is normal inside `K`.
theorem subgroup_intersection_normal_in_right[G: Group](h: Subgroup[G], k: Subgroup[G], g: G, x: G) {
    is_normal_subgroup(h) and k.contains(g) and h.intersection(k).contains(x) implies
    h.intersection(k).contains(g * x * g.inverse)
} by {
    if is_normal_subgroup(h) and k.contains(g) and h.intersection(k).contains(x) {
        subgroup_intersection_subset_left(h, k, x)
        subgroup_intersection_subset_right(h, k, x)
        subgroup_inv_mem(k, g)
        subgroup_mul_mem(k, g, x)
        subgroup_mul_mem(k, g * x, g.inverse)
        k.contains(g * x * g.inverse)
        normal_conjugate_mem(h, g, x)
        h.contains(g * x * g.inverse)
        subgroup_intersection_contains_eq(h, k, g * x * g.inverse)
        h.intersection(k).contains(g * x * g.inverse)
    }
}

// ==== The projection to G/K restricted to H ====

/// The projection to `G/K` restricted to `H` preserves multiplication.
theorem second_iso_project_mul[G: Group](h: Subgroup[G], k: Subgroup[G], a: G, b: G) {
    is_normal_subgroup(k) and h.contains(a) and h.contains(b) implies
    normal_subgroup_quotient_project(k, a * b) =
        normal_subgroup_quotient_mul(k,
            normal_subgroup_quotient_project(k, a),
            normal_subgroup_quotient_project(k, b))
} by {
    if is_normal_subgroup(k) and h.contains(a) and h.contains(b) {
        normal_subgroup_quotient_project_mul(k, a, b)
        normal_subgroup_quotient_project(k, a * b) =
            normal_subgroup_quotient_mul(k,
                normal_subgroup_quotient_project(k, a),
                normal_subgroup_quotient_project(k, b))
    }
}

/// The projection to `G/K` restricted to `H` preserves inverses.
theorem second_iso_project_inverse[G: Group](h: Subgroup[G], k: Subgroup[G], a: G) {
    is_normal_subgroup(k) and h.contains(a) implies
    normal_subgroup_quotient_project(k, a.inverse) =
        normal_subgroup_quotient_inverse(k, normal_subgroup_quotient_project(k, a))
} by {
    if is_normal_subgroup(k) and h.contains(a) {
        normal_subgroup_quotient_project_inverse(k, a)
        normal_subgroup_quotient_project(k, a.inverse) =
            normal_subgroup_quotient_inverse(k, normal_subgroup_quotient_project(k, a))
    }
}

/// The kernel of the projection to `G/K` restricted to `H` is exactly `H ∩ K`:
/// an element of `H` projects to the quotient identity exactly when it lies in
/// both subgroups.
theorem second_iso_kernel_iff[G: Group](h: Subgroup[G], k: Subgroup[G], a: G) {
    h.contains(a) implies
    (normal_subgroup_quotient_project(k, a) = normal_subgroup_quotient_one(k)) =
        h.intersection(k).contains(a)
} by {
    if h.contains(a) {
        normal_subgroup_quotient_project_eq_one_iff_contains(k, a)
        (normal_subgroup_quotient_project(k, a) = normal_subgroup_quotient_one(k)) =
            k.contains(a)
        subgroup_intersection_contains_eq(h, k, a)
        h.intersection(k).contains(a) = (h.contains(a) and k.contains(a))
        h.contains(a) and k.contains(a) = k.contains(a)
        h.intersection(k).contains(a) = k.contains(a)
        (normal_subgroup_quotient_project(k, a) = normal_subgroup_quotient_one(k)) =
            h.intersection(k).contains(a)
    }
}

/// The kernel relation of the projection to `G/K` restricted to `H` is exactly
/// the relation of `H ∩ K` on `H`: two elements of `H` project to equal
/// `K`-cosets exactly when they differ by an element of `H ∩ K`.
theorem second_iso_kernel_rel[G: Group](h: Subgroup[G], k: Subgroup[G], a: G, b: G) {
    h.contains(a) and h.contains(b) implies
    (normal_subgroup_quotient_project(k, a) = normal_subgroup_quotient_project(k, b)) =
        normal_subgroup_rel(h.intersection(k), a, b)
} by {
    if h.contains(a) and h.contains(b) {
        normal_subgroup_quotient_project_eq_iff_rel(k, a, b)
        (normal_subgroup_quotient_project(k, a) = normal_subgroup_quotient_project(k, b)) =
            normal_subgroup_rel(k, a, b)
        normal_subgroup_rel(k, a, b) = k.contains(a * b.inverse)
        subgroup_inv_mem(h, b)
        subgroup_mul_mem(h, a, b.inverse)
        h.contains(a * b.inverse)
        subgroup_intersection_contains_eq(h, k, a * b.inverse)
        h.intersection(k).contains(a * b.inverse) = (h.contains(a * b.inverse) and k.contains(a * b.inverse))
        h.contains(a * b.inverse) and k.contains(a * b.inverse) = k.contains(a * b.inverse)
        h.intersection(k).contains(a * b.inverse) = k.contains(a * b.inverse)
        normal_subgroup_rel(h.intersection(k), a, b) = h.intersection(k).contains(a * b.inverse)
        (normal_subgroup_quotient_project(k, a) = normal_subgroup_quotient_project(k, b)) =
            normal_subgroup_rel(h.intersection(k), a, b)
    }
}

/// The induced map `H/(H ∩ K) -> G/K` is injective: two elements of `H`
/// project to equal `K`-cosets exactly when they are equal in the quotient of
/// `H` by `H ∩ K`.
theorem second_iso_induced_injective[G: Group](h: Subgroup[G], k: Subgroup[G], a: G, b: G) {
    h.contains(a) and h.contains(b) implies
    (normal_subgroup_quotient_project(k, a) = normal_subgroup_quotient_project(k, b)) =
        (normal_subgroup_quotient_project(h.intersection(k), a) =
            normal_subgroup_quotient_project(h.intersection(k), b))
} by {
    if h.contains(a) and h.contains(b) {
        second_iso_kernel_rel(h, k, a, b)
        normal_subgroup_quotient_project_eq_iff_rel(h.intersection(k), a, b)
        (normal_subgroup_quotient_project(h.intersection(k), a) =
            normal_subgroup_quotient_project(h.intersection(k), b)) =
            normal_subgroup_rel(h.intersection(k), a, b)
        (normal_subgroup_quotient_project(k, a) = normal_subgroup_quotient_project(k, b)) =
            (normal_subgroup_quotient_project(h.intersection(k), a) =
                normal_subgroup_quotient_project(h.intersection(k), b))
    }
}

// ==== The image is HK/K ====

/// A quotient element is the projection of an element of `H` exactly when it is
/// the projection of an element of the product `HK`: the image of the
/// restricted projection is the set of `K`-cosets represented in `H`, i.e.
/// `HK/K`.
theorem second_iso_image_iff_product[G: Group](h: Subgroup[G], k: Subgroup[G], q: QuotientOver[G]) {
    is_normal_subgroup(k) implies
    (exists(a: G) {
        h.contains(a) and normal_subgroup_quotient_project(k, a) = q
    }) =
    (exists(x: G) {
        subgroup_product_contains(h, k, x) and normal_subgroup_quotient_project(k, x) = q
    })
} by {
    if is_normal_subgroup(k) {
        if exists(a: G) {
            h.contains(a) and normal_subgroup_quotient_project(k, a) = q
        } {
            let a: G satisfy {
                h.contains(a) and normal_subgroup_quotient_project(k, a) = q
            }
            subgroup_contains_identity(k)
            a = a * G.1
            subgroup_product_contains(h, k, a)
            exists(x: G) {
                subgroup_product_contains(h, k, x) and normal_subgroup_quotient_project(k, x) = q
            }
        }
        if exists(x: G) {
            subgroup_product_contains(h, k, x) and normal_subgroup_quotient_project(k, x) = q
        } {
            let x: G satisfy {
                subgroup_product_contains(h, k, x) and normal_subgroup_quotient_project(k, x) = q
            }
            let (a: G, b: G) satisfy {
                h.contains(a) and k.contains(b) and x = a * b
            }
            normal_subgroup_quotient_project_mul(k, a, b)
            normal_subgroup_quotient_project(k, a * b) =
                normal_subgroup_quotient_mul(k,
                    normal_subgroup_quotient_project(k, a),
                    normal_subgroup_quotient_project(k, b))
            normal_subgroup_quotient_project_eq_one_iff_contains(k, b)
            normal_subgroup_quotient_project(k, b) = normal_subgroup_quotient_one(k)
            normal_subgroup_quotient_project_eq_mk(k, a)
            normal_subgroup_quotient_mul_one(k, a)
            normal_subgroup_quotient_mul(k,
                normal_subgroup_quotient_project(k, a),
                normal_subgroup_quotient_one(k)) =
                normal_subgroup_quotient_project(k, a)
            normal_subgroup_quotient_project(k, a * b) =
                normal_subgroup_quotient_project(k, a)
            x = a * b
            normal_subgroup_quotient_project(k, x) = normal_subgroup_quotient_project(k, a * b)
            normal_subgroup_quotient_project(k, x) = normal_subgroup_quotient_project(k, a)
            normal_subgroup_quotient_project(k, a) = q
            exists(a1: G) {
                h.contains(a1) and normal_subgroup_quotient_project(k, a1) = q
            }
        }
        (exists(a: G) {
            h.contains(a) and normal_subgroup_quotient_project(k, a) = q
        }) =
        (exists(x: G) {
            subgroup_product_contains(h, k, x) and normal_subgroup_quotient_project(k, x) = q
        })
    }
}

// ==== The dual statement: H normal, projection to G/H restricted to K ====

/// The kernel of the projection to `G/H` restricted to `K` is exactly
/// `H ∩ K`: an element of `K` projects to the quotient identity exactly when
/// it lies in both subgroups.
theorem second_iso_dual_kernel_iff[G: Group](h: Subgroup[G], k: Subgroup[G], b: G) {
    k.contains(b) implies
    (normal_subgroup_quotient_project(h, b) = normal_subgroup_quotient_one(h)) =
        h.intersection(k).contains(b)
} by {
    if k.contains(b) {
        if normal_subgroup_quotient_project(h, b) = normal_subgroup_quotient_one(h) {
            normal_subgroup_quotient_project_eq_one_iff_contains(h, b)
            (normal_subgroup_quotient_project(h, b) = normal_subgroup_quotient_one(h)) =
                h.contains(b)
            h.contains(b)
            k.contains(b)
            subgroup_intersection_contains_eq(h, k, b)
            h.intersection(k).contains(b) = (h.contains(b) and k.contains(b))
            h.contains(b) and k.contains(b)
            h.intersection(k).contains(b)
        }
        if h.intersection(k).contains(b) {
            subgroup_intersection_contains_eq(h, k, b)
            h.intersection(k).contains(b) = (h.contains(b) and k.contains(b))
            h.contains(b) and k.contains(b)
            h.contains(b)
            normal_subgroup_quotient_project_eq_one_iff_contains(h, b)
            (normal_subgroup_quotient_project(h, b) = normal_subgroup_quotient_one(h)) =
                h.contains(b)
            normal_subgroup_quotient_project(h, b) = normal_subgroup_quotient_one(h)
        }
        (normal_subgroup_quotient_project(h, b) = normal_subgroup_quotient_one(h)) =
            h.intersection(k).contains(b)
    }
}

/// The kernel relation of the projection to `G/H` restricted to `K` is exactly
/// the relation of `H ∩ K` on `K`.
theorem second_iso_dual_kernel_rel[G: Group](h: Subgroup[G], k: Subgroup[G], a: G, b: G) {
    k.contains(a) and k.contains(b) implies
    (normal_subgroup_quotient_project(h, a) = normal_subgroup_quotient_project(h, b)) =
        normal_subgroup_rel(h.intersection(k), a, b)
} by {
    if k.contains(a) and k.contains(b) {
        if normal_subgroup_quotient_project(h, a) = normal_subgroup_quotient_project(h, b) {
            normal_subgroup_quotient_project_eq_iff_rel(h, a, b)
            (normal_subgroup_quotient_project(h, a) = normal_subgroup_quotient_project(h, b)) =
                normal_subgroup_rel(h, a, b)
            normal_subgroup_rel(h, a, b)
            normal_subgroup_rel(h, a, b) = h.contains(a * b.inverse)
            h.contains(a * b.inverse)
            subgroup_inv_mem(k, b)
            subgroup_mul_mem(k, a, b.inverse)
            k.contains(a * b.inverse)
            subgroup_intersection_contains_eq(h, k, a * b.inverse)
            h.intersection(k).contains(a * b.inverse) =
                (h.contains(a * b.inverse) and k.contains(a * b.inverse))
            h.contains(a * b.inverse) and k.contains(a * b.inverse)
            h.intersection(k).contains(a * b.inverse)
            normal_subgroup_rel(h.intersection(k), a, b) =
                h.intersection(k).contains(a * b.inverse)
            normal_subgroup_rel(h.intersection(k), a, b)
        }
        if normal_subgroup_rel(h.intersection(k), a, b) {
            normal_subgroup_rel(h.intersection(k), a, b) =
                h.intersection(k).contains(a * b.inverse)
            h.intersection(k).contains(a * b.inverse)
            subgroup_intersection_contains_eq(h, k, a * b.inverse)
            h.intersection(k).contains(a * b.inverse) =
                (h.contains(a * b.inverse) and k.contains(a * b.inverse))
            h.contains(a * b.inverse) and k.contains(a * b.inverse)
            h.contains(a * b.inverse)
            normal_subgroup_rel(h, a, b) = h.contains(a * b.inverse)
            normal_subgroup_rel(h, a, b)
            normal_subgroup_quotient_project_eq_of_rel(h, a, b)
            normal_subgroup_quotient_project(h, a) = normal_subgroup_quotient_project(h, b)
        }
        (normal_subgroup_quotient_project(h, a) = normal_subgroup_quotient_project(h, b)) =
            normal_subgroup_rel(h.intersection(k), a, b)
    }
}

/// The induced map `K/(H ∩ K) -> G/H` is injective: two elements of `K`
/// project to equal `H`-cosets exactly when they are equal in the quotient of
/// `K` by `H ∩ K`.
theorem second_iso_dual_induced_injective[G: Group](h: Subgroup[G], k: Subgroup[G], a: G, b: G) {
    k.contains(a) and k.contains(b) implies
    (normal_subgroup_quotient_project(h, a) = normal_subgroup_quotient_project(h, b)) =
        (normal_subgroup_quotient_project(h.intersection(k), a) =
            normal_subgroup_quotient_project(h.intersection(k), b))
} by {
    if k.contains(a) and k.contains(b) {
        second_iso_dual_kernel_rel(h, k, a, b)
        normal_subgroup_quotient_project_eq_iff_rel(h.intersection(k), a, b)
        (normal_subgroup_quotient_project(h.intersection(k), a) =
            normal_subgroup_quotient_project(h.intersection(k), b)) =
            normal_subgroup_rel(h.intersection(k), a, b)
        (normal_subgroup_quotient_project(h, a) = normal_subgroup_quotient_project(h, b)) =
            (normal_subgroup_quotient_project(h.intersection(k), a) =
                normal_subgroup_quotient_project(h.intersection(k), b))
    }
}

/// A quotient element is the projection of an element of `K` exactly when it is
/// the projection of an element of the product `HK`: the image of the
/// restricted projection is the set of `H`-cosets represented in `K`, i.e.
/// `HK/H`.
theorem second_iso_dual_image_iff_product[G: Group](h: Subgroup[G], k: Subgroup[G], q: QuotientOver[G]) {
    is_normal_subgroup(h) implies
    (exists(b: G) {
        k.contains(b) and normal_subgroup_quotient_project(h, b) = q
    }) =
    (exists(x: G) {
        subgroup_product_contains(h, k, x) and normal_subgroup_quotient_project(h, x) = q
    })
} by {
    if is_normal_subgroup(h) {
        if exists(b: G) {
            k.contains(b) and normal_subgroup_quotient_project(h, b) = q
        } {
            let b: G satisfy {
                k.contains(b) and normal_subgroup_quotient_project(h, b) = q
            }
            subgroup_contains_identity(h)
            b = G.1 * b
            subgroup_product_contains(h, k, b)
            exists(x: G) {
                subgroup_product_contains(h, k, x) and normal_subgroup_quotient_project(h, x) = q
            }
        }
        if exists(x: G) {
            subgroup_product_contains(h, k, x) and normal_subgroup_quotient_project(h, x) = q
        } {
            let x: G satisfy {
                subgroup_product_contains(h, k, x) and normal_subgroup_quotient_project(h, x) = q
            }
            let (a: G, b: G) satisfy {
                h.contains(a) and k.contains(b) and x = a * b
            }
            normal_subgroup_quotient_project_mul(h, a, b)
            normal_subgroup_quotient_project(h, a * b) =
                normal_subgroup_quotient_mul(h,
                    normal_subgroup_quotient_project(h, a),
                    normal_subgroup_quotient_project(h, b))
            normal_subgroup_quotient_project_eq_one_iff_contains(h, a)
            normal_subgroup_quotient_project(h, a) = normal_subgroup_quotient_one(h)
            normal_subgroup_quotient_project_eq_mk(h, b)
            normal_subgroup_quotient_one_mul(h, b)
            normal_subgroup_quotient_mul(h,
                normal_subgroup_quotient_one(h),
                normal_subgroup_quotient_project(h, b)) =
                normal_subgroup_quotient_project(h, b)
            normal_subgroup_quotient_project(h, a * b) =
                normal_subgroup_quotient_project(h, b)
            x = a * b
            normal_subgroup_quotient_project(h, x) = normal_subgroup_quotient_project(h, a * b)
            normal_subgroup_quotient_project(h, x) = normal_subgroup_quotient_project(h, b)
            normal_subgroup_quotient_project(h, b) = q
            exists(b1: G) {
                k.contains(b1) and normal_subgroup_quotient_project(h, b1) = q
            }
        }
        (exists(b: G) {
            k.contains(b) and normal_subgroup_quotient_project(h, b) = q
        }) =
        (exists(x: G) {
            subgroup_product_contains(h, k, x) and normal_subgroup_quotient_project(h, x) = q
        })
    }
}
