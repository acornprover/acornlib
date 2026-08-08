/// Bundled bijection API for group-action maps.

from algebra.group import Group, inverse_left
from algebra.group_action import MulAction, action_bijection, action_bijection_apply,
    action_bijection_one_map, action_bijection_mul_map,
    mul_action_apply_inv_apply
from data.basic.functions import Inhabited, compose, identity_bijection, identity_bijection_map,
    compose_bijection, compose_bijection_map, inverse_bijection,
    bijection_apply_inverse, bijection_ext, bijection_map_is_injective,
    injective_fn_eq, function_extensionality

/// The action bijection of the identity group element is the identity bijection.
theorem action_bijection_identity[G: Group, X](a: MulAction[G, X], anchor: X) {
    action_bijection(a, G.1) = identity_bijection(anchor)
} by {
    action_bijection_one_map(a)
    identity_bijection_map(anchor)
    bijection_ext(action_bijection(a, G.1), identity_bijection(anchor))
}

/// Composition of action bijections is the action bijection of the product element.
theorem compose_action_bijection[G: Group, X](a: MulAction[G, X], g: G, h: G) {
    compose_bijection(action_bijection(a, g), action_bijection(a, h)) =
        action_bijection(a, g * h)
} by {
    compose_bijection_map(action_bijection(a, g), action_bijection(a, h))
    action_bijection_mul_map(a, g, h)
    bijection_ext(compose_bijection(action_bijection(a, g), action_bijection(a, h)),
        action_bijection(a, g * h))
}

/// Composing the inverse action bijection on the left gives the identity bijection.
theorem action_bijection_inverse_left[G: Group, X](a: MulAction[G, X], g: G, anchor: X) {
    compose_bijection(action_bijection(a, g.inverse), action_bijection(a, g)) =
        identity_bijection(anchor)
} by {
    compose_action_bijection(a, g.inverse, g)
    inverse_left(g)
    action_bijection_identity(a, anchor)
}

/// Composing the inverse action bijection on the right gives the identity bijection.
theorem action_bijection_inverse_right[G: Group, X](a: MulAction[G, X], g: G, anchor: X) {
    compose_bijection(action_bijection(a, g), action_bijection(a, g.inverse)) =
        identity_bijection(anchor)
} by {
    compose_action_bijection(a, g, g.inverse)
    action_bijection_identity(a, anchor)
}

/// The inverse bijection of an action bijection evaluates as the inverse group element acts.
theorem inverse_bijection_action_bijection_apply[G: Group, X: Inhabited](
    a: MulAction[G, X], g: G, x: X
) {
    inverse_bijection(action_bijection(a, g)).map(x) = action_bijection(a, g.inverse).map(x)
} by {
    let e = action_bijection(a, g)
    let h = action_bijection(a, g.inverse)
    bijection_apply_inverse(e, x)
    action_bijection_apply(a, g.inverse, x)
    action_bijection_apply(a, g, h.map(x))
    mul_action_apply_inv_apply(a, g, x)
    bijection_map_is_injective(e)
    injective_fn_eq(e.map, inverse_bijection(e).map(x), h.map(x))
    inverse_bijection(e).map(x) = h.map(x)
}

/// The inverse action bijection has the same underlying map as the action bijection of the inverse group element.
theorem inverse_bijection_action_bijection_map[G: Group, X: Inhabited](a: MulAction[G, X], g: G) {
    inverse_bijection(action_bijection(a, g)).map = action_bijection(a, g.inverse).map
} by {
    forall(x: X) {
        inverse_bijection_action_bijection_apply(a, g, x)
    }
    function_extensionality(inverse_bijection(action_bijection(a, g)).map,
        action_bijection(a, g.inverse).map)
}

/// The inverse of an action bijection is the action bijection of the inverse group element.
theorem inverse_bijection_action_bijection[G: Group, X: Inhabited](a: MulAction[G, X], g: G) {
    inverse_bijection(action_bijection(a, g)) = action_bijection(a, g.inverse)
} by {
    inverse_bijection_action_bijection_map(a, g)
    bijection_ext(inverse_bijection(action_bijection(a, g)), action_bijection(a, g.inverse))
}
