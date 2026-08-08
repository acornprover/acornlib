from algebra.group import Group
from pair import Pair, pair_ext
from algebra.group_action import MulAction, is_mul_action, action_identity_constraint,
    action_mul_constraint, mul_action_one, mul_action_mul, orbit, orbit_contains_witness,
    orbit_contains_action, orbit_eq_of_contains, is_equivariant_map, equivariant_map_apply, ActionHom,
    is_transitive_action, transitive_action_orbit_contains, is_faithful_action,
    faithful_action_eq_of_pointwise_eq
from algebra.group_action import equivariant_map_maps_orbit_contains,
    equivariant_map_maps_stabilizer_contains
from algebra.group_action import stabilizer, stabilizer_fixes, stabilizer_contains_of_fixes
from algebra.subgroup import subgroup_ext, subgroup_intersection_contains_eq

/// The componentwise action on a product of two actions of the same group.
define product_act[G: Group, X, Y](a: MulAction[G, X], b: MulAction[G, Y],
    g: G, p: Pair[X, Y]) -> Pair[X, Y] {
    Pair.new(a.act(g, p.first), b.act(g, p.second))
}

/// The first coordinate of the product action.
theorem product_act_first[G: Group, X, Y](a: MulAction[G, X], b: MulAction[G, Y],
    g: G, p: Pair[X, Y]) {
    product_act(a, b, g, p).first = a.act(g, p.first)
}

/// The second coordinate of the product action.
theorem product_act_second[G: Group, X, Y](a: MulAction[G, X], b: MulAction[G, Y],
    g: G, p: Pair[X, Y]) {
    product_act(a, b, g, p).second = b.act(g, p.second)
}

/// The product action fixes a product at the identity element.
theorem product_act_one[G: Group, X, Y](a: MulAction[G, X], b: MulAction[G, Y], p: Pair[X, Y]) {
    product_act(a, b, G.1, p) = p
} by {
    let lhs = product_act(a, b, G.1, p)
    lhs.first = a.act(G.1, p.first)
    mul_action_one(a, p.first)
    lhs.first = p.first
    lhs.second = b.act(G.1, p.second)
    mul_action_one(b, p.second)
    lhs.second = p.second
    pair_ext(lhs, p)
}

/// The product action satisfies the identity constraint.
theorem product_act_identity_constraint[G: Group, X, Y](a: MulAction[G, X], b: MulAction[G, Y]) {
    action_identity_constraint(product_act(a, b))
} by {
    action_identity_constraint(product_act(a, b)) = forall(p: Pair[X, Y]) {
        product_act(a, b, G.1, p) = p
    }
    forall(p: Pair[X, Y]) {
        product_act_one(a, b, p)
    }
}

/// The product action distributes over multiplication of group elements.
theorem product_act_mul_apply[G: Group, X, Y](a: MulAction[G, X], b: MulAction[G, Y],
    g: G, h: G, p: Pair[X, Y]) {
    product_act(a, b, g * h, p) = product_act(a, b, g, product_act(a, b, h, p))
} by {
    let lhs = product_act(a, b, g * h, p)
    let rhs = product_act(a, b, g, product_act(a, b, h, p))
    lhs.first = a.act(g * h, p.first)
    mul_action_mul(a, g, h, p.first)
    lhs.first = a.act(g, a.act(h, p.first))
    product_act(a, b, h, p).first = a.act(h, p.first)
    rhs.first = a.act(g, product_act(a, b, h, p).first)
    rhs.first = a.act(g, a.act(h, p.first))
    lhs.first = rhs.first
    lhs.second = b.act(g * h, p.second)
    mul_action_mul(b, g, h, p.second)
    lhs.second = b.act(g, b.act(h, p.second))
    product_act(a, b, h, p).second = b.act(h, p.second)
    rhs.second = b.act(g, product_act(a, b, h, p).second)
    rhs.second = b.act(g, b.act(h, p.second))
    lhs.second = rhs.second
    pair_ext(lhs, rhs)
}

/// The product action satisfies the multiplication constraint.
theorem product_act_mul_constraint[G: Group, X, Y](a: MulAction[G, X], b: MulAction[G, Y]) {
    action_mul_constraint(product_act(a, b))
} by {
    action_mul_constraint(product_act(a, b)) = forall(g: G, h: G, p: Pair[X, Y]) {
        product_act(a, b, g * h, p) = product_act(a, b, g, product_act(a, b, h, p))
    }
    forall(g: G, h: G, p: Pair[X, Y]) {
        product_act_mul_apply(a, b, g, h, p)
    }
}

/// The componentwise product action is a left action.
theorem product_act_is_mul_action[G: Group, X, Y](a: MulAction[G, X], b: MulAction[G, Y]) {
    is_mul_action(product_act(a, b))
} by {
    product_act_mul_constraint(a, b)
    product_act_identity_constraint(a, b)
}

/// The bundled componentwise product action.
let product_action[G: Group, X, Y](a: MulAction[G, X], b: MulAction[G, Y]) -> result: MulAction[G, Pair[X, Y]] satisfy {
    MulAction.new(product_act(a, b)) = Option.some(result)
} by {
    product_act_is_mul_action(a, b)
}

/// The bundled product action evaluates componentwise.
theorem product_action_act[G: Group, X, Y](a: MulAction[G, X], b: MulAction[G, Y],
    g: G, p: Pair[X, Y]) {
    product_action(a, b).act(g, p) = Pair.new(a.act(g, p.first), b.act(g, p.second))
} by {
    product_action(a, b).act(g, p) = product_act(a, b, g, p)
}

/// The first coordinate of the bundled product action.
theorem product_action_first[G: Group, X, Y](a: MulAction[G, X], b: MulAction[G, Y],
    g: G, p: Pair[X, Y]) {
    product_action(a, b).act(g, p).first = a.act(g, p.first)
} by {
    product_action_act(a, b, g, p)
}

/// The second coordinate of the bundled product action.
theorem product_action_second[G: Group, X, Y](a: MulAction[G, X], b: MulAction[G, Y],
    g: G, p: Pair[X, Y]) {
    product_action(a, b).act(g, p).second = b.act(g, p.second)
} by {
    product_action_act(a, b, g, p)
}

/// The first projection map from a product.
define product_first_projection[X, Y](p: Pair[X, Y]) -> X {
    p.first
}

/// The second projection map from a product.
define product_second_projection[X, Y](p: Pair[X, Y]) -> Y {
    p.second
}

/// The first projection is equivariant from the product action to the first factor.
theorem product_action_first_projection_is_equivariant[G: Group, X, Y](
    a: MulAction[G, X], b: MulAction[G, Y]) {
    is_equivariant_map(product_action(a, b), a, Pair.first[X, Y])
} by {
    forall(g: G, p: Pair[X, Y]) {
        Pair.first[X, Y](product_action(a, b).act(g, p)) = product_action(a, b).act(g, p).first
        product_action_first(a, b, g, p)
        Pair.first[X, Y](p) = p.first
        Pair.first[X, Y](product_action(a, b).act(g, p)) = a.act(g, Pair.first[X, Y](p))
    }
}

/// The named first projection is equivariant from the product action to the first factor.
theorem product_action_product_first_projection_is_equivariant[G: Group, X, Y](
    a: MulAction[G, X], b: MulAction[G, Y]) {
    is_equivariant_map(product_action(a, b), a, product_first_projection[X, Y])
} by {
    forall(g: G, p: Pair[X, Y]) {
        product_first_projection[X, Y](product_action(a, b).act(g, p)) =
            product_action(a, b).act(g, p).first
        product_action_first(a, b, g, p)
        product_first_projection[X, Y](p) = p.first
        product_first_projection[X, Y](product_action(a, b).act(g, p)) =
            a.act(g, product_first_projection[X, Y](p))
    }
}

/// The second projection is equivariant from the product action to the second factor.
theorem product_action_second_projection_is_equivariant[G: Group, X, Y](
    a: MulAction[G, X], b: MulAction[G, Y]) {
    is_equivariant_map(product_action(a, b), b, Pair.second[X, Y])
} by {
    forall(g: G, p: Pair[X, Y]) {
        Pair.second[X, Y](product_action(a, b).act(g, p)) = product_action(a, b).act(g, p).second
        product_action_second(a, b, g, p)
        Pair.second[X, Y](p) = p.second
        Pair.second[X, Y](product_action(a, b).act(g, p)) = b.act(g, Pair.second[X, Y](p))
    }
}

/// The named second projection is equivariant from the product action to the second factor.
theorem product_action_product_second_projection_is_equivariant[G: Group, X, Y](
    a: MulAction[G, X], b: MulAction[G, Y]) {
    is_equivariant_map(product_action(a, b), b, product_second_projection[X, Y])
} by {
    forall(g: G, p: Pair[X, Y]) {
        product_second_projection[X, Y](product_action(a, b).act(g, p)) =
            product_action(a, b).act(g, p).second
        product_action_second(a, b, g, p)
        product_second_projection[X, Y](p) = p.second
        product_second_projection[X, Y](product_action(a, b).act(g, p)) =
            b.act(g, product_second_projection[X, Y](p))
    }
}

/// The first projection as an action homomorphism.
let product_action_first_projection_hom[G: Group, X, Y](a: MulAction[G, X], b: MulAction[G, Y]) -> result: ActionHom[G, Pair[X, Y], X] satisfy {
    ActionHom[G, Pair[X, Y], X].new(product_action(a, b), a, product_first_projection[X, Y]) =
        Option.some(result)
} by {
    product_action_product_first_projection_is_equivariant(a, b)
}

/// The first projection homomorphism has the product action as source.
theorem product_action_first_projection_hom_src[G: Group, X, Y](
    a: MulAction[G, X], b: MulAction[G, Y]) {
    product_action_first_projection_hom(a, b).src = product_action(a, b)
}

/// The first projection homomorphism has the first factor action as target.
theorem product_action_first_projection_hom_dst[G: Group, X, Y](
    a: MulAction[G, X], b: MulAction[G, Y]) {
    product_action_first_projection_hom(a, b).dst = a
}

/// The second projection as an action homomorphism.
let product_action_second_projection_hom[G: Group, X, Y](a: MulAction[G, X], b: MulAction[G, Y]) -> result: ActionHom[G, Pair[X, Y], Y] satisfy {
    ActionHom[G, Pair[X, Y], Y].new(product_action(a, b), b, product_second_projection[X, Y]) =
        Option.some(result)
} by {
    product_action_product_second_projection_is_equivariant(a, b)
}

/// The second projection homomorphism has the product action as source.
theorem product_action_second_projection_hom_src[G: Group, X, Y](
    a: MulAction[G, X], b: MulAction[G, Y]) {
    product_action_second_projection_hom(a, b).src = product_action(a, b)
}

/// The second projection homomorphism has the second factor action as target.
theorem product_action_second_projection_hom_dst[G: Group, X, Y](
    a: MulAction[G, X], b: MulAction[G, Y]) {
    product_action_second_projection_hom(a, b).dst = b
}

/// Product-action orbit membership gives orbit membership in each coordinate.
theorem product_action_orbit_projects[G: Group, X, Y](a: MulAction[G, X], b: MulAction[G, Y],
    p: Pair[X, Y], q: Pair[X, Y]) {
    orbit(product_action(a, b), p).contains(q) implies
        orbit(a, p.first).contains(q.first) and orbit(b, p.second).contains(q.second)
} by {
    if orbit(product_action(a, b), p).contains(q) {
        orbit_contains_witness(product_action(a, b), p, q)
        let g: G satisfy {
            q = product_action(a, b).act(g, p)
        }
        product_action_first(a, b, g, p)
        product_action_second(a, b, g, p)
        q.first = a.act(g, p.first)
        q.second = b.act(g, p.second)
        orbit_contains_action(a, p.first, g)
        orbit_contains_action(b, p.second, g)
        orbit(a, p.first).contains(q.first)
        orbit(b, p.second).contains(q.second)
    }
}

/// Coordinate orbit witnesses with the same group element give product-action orbit membership.
theorem product_action_orbit_contains_of_same_witness[G: Group, X, Y](
    a: MulAction[G, X], b: MulAction[G, Y], p: Pair[X, Y], q: Pair[X, Y], g: G) {
    q.first = a.act(g, p.first) and q.second = b.act(g, p.second) implies
        orbit(product_action(a, b), p).contains(q)
} by {
    if q.first = a.act(g, p.first) and q.second = b.act(g, p.second) {
        product_action_act(a, b, g, p)
        product_action(a, b).act(g, p).first = q.first
        product_action(a, b).act(g, p).second = q.second
        pair_ext(product_action(a, b).act(g, p), q)
        q = product_action(a, b).act(g, p)
        orbit_contains_action(product_action(a, b), p, g)
        orbit(product_action(a, b), p).contains(q)
    }
}

/// Points in the same product-action orbit have the same product-action orbit.
theorem product_action_orbit_eq_of_contains[G: Group, X, Y](
    a: MulAction[G, X], b: MulAction[G, Y], p: Pair[X, Y], q: Pair[X, Y]) {
    orbit(product_action(a, b), p).contains(q) implies
        orbit(product_action(a, b), p) = orbit(product_action(a, b), q)
} by {
    if orbit(product_action(a, b), p).contains(q) {
        orbit_eq_of_contains(product_action(a, b), p, q)
    }
}

/// Product-action orbit membership gives equality of first-coordinate orbits.
theorem product_action_first_orbit_eq_of_contains[G: Group, X, Y](
    a: MulAction[G, X], b: MulAction[G, Y], p: Pair[X, Y], q: Pair[X, Y]) {
    orbit(product_action(a, b), p).contains(q) implies orbit(a, p.first) = orbit(a, q.first)
} by {
    if orbit(product_action(a, b), p).contains(q) {
        product_action_orbit_projects(a, b, p, q)
        orbit(a, p.first).contains(q.first)
        orbit_eq_of_contains(a, p.first, q.first)
    }
}

/// Product-action orbit membership gives equality of second-coordinate orbits.
theorem product_action_second_orbit_eq_of_contains[G: Group, X, Y](
    a: MulAction[G, X], b: MulAction[G, Y], p: Pair[X, Y], q: Pair[X, Y]) {
    orbit(product_action(a, b), p).contains(q) implies orbit(b, p.second) = orbit(b, q.second)
} by {
    if orbit(product_action(a, b), p).contains(q) {
        product_action_orbit_projects(a, b, p, q)
        orbit(b, p.second).contains(q.second)
        orbit_eq_of_contains(b, p.second, q.second)
    }
}

/// A product-action stabilizer member fixes the first coordinate.
theorem product_action_stabilizer_fixes_first[G: Group, X, Y](
    a: MulAction[G, X], b: MulAction[G, Y], p: Pair[X, Y], g: G) {
    stabilizer(product_action(a, b), p).contains(g) implies a.act(g, p.first) = p.first
} by {
    if stabilizer(product_action(a, b), p).contains(g) {
        stabilizer_fixes(product_action(a, b), p, g)
        product_action_first(a, b, g, p)
        product_action(a, b).act(g, p) = p
        product_action(a, b).act(g, p).first = p.first
        a.act(g, p.first) = p.first
    }
}

/// A product-action stabilizer member fixes the second coordinate.
theorem product_action_stabilizer_fixes_second[G: Group, X, Y](
    a: MulAction[G, X], b: MulAction[G, Y], p: Pair[X, Y], g: G) {
    stabilizer(product_action(a, b), p).contains(g) implies b.act(g, p.second) = p.second
} by {
    if stabilizer(product_action(a, b), p).contains(g) {
        stabilizer_fixes(product_action(a, b), p, g)
        product_action_second(a, b, g, p)
        product_action(a, b).act(g, p) = p
        product_action(a, b).act(g, p).second = p.second
        b.act(g, p.second) = p.second
    }
}

/// A product-action stabilizer member belongs to the first-coordinate stabilizer.
theorem product_action_stabilizer_subset_first[G: Group, X, Y](
    a: MulAction[G, X], b: MulAction[G, Y], p: Pair[X, Y], g: G) {
    stabilizer(product_action(a, b), p).contains(g) implies stabilizer(a, p.first).contains(g)
} by {
    if stabilizer(product_action(a, b), p).contains(g) {
        product_action_stabilizer_fixes_first(a, b, p, g)
        stabilizer_contains_of_fixes(a, p.first, g)
        stabilizer(a, p.first).contains(g)
    }
}

/// A product-action stabilizer member belongs to the second-coordinate stabilizer.
theorem product_action_stabilizer_subset_second[G: Group, X, Y](
    a: MulAction[G, X], b: MulAction[G, Y], p: Pair[X, Y], g: G) {
    stabilizer(product_action(a, b), p).contains(g) implies stabilizer(b, p.second).contains(g)
} by {
    if stabilizer(product_action(a, b), p).contains(g) {
        product_action_stabilizer_fixes_second(a, b, p, g)
        stabilizer_contains_of_fixes(b, p.second, g)
        stabilizer(b, p.second).contains(g)
    }
}

/// Common coordinate stabilizer membership gives product-action stabilizer membership.
theorem product_action_stabilizer_contains_of_coordinates[G: Group, X, Y](
    a: MulAction[G, X], b: MulAction[G, Y], p: Pair[X, Y], g: G) {
    stabilizer(a, p.first).contains(g) and stabilizer(b, p.second).contains(g) implies
        stabilizer(product_action(a, b), p).contains(g)
} by {
    if stabilizer(a, p.first).contains(g) and stabilizer(b, p.second).contains(g) {
        stabilizer_fixes(a, p.first, g)
        stabilizer_fixes(b, p.second, g)
        product_action_act(a, b, g, p)
        product_action(a, b).act(g, p).first = a.act(g, p.first)
        product_action(a, b).act(g, p).second = b.act(g, p.second)
        product_action(a, b).act(g, p).first = p.first
        product_action(a, b).act(g, p).second = p.second
        pair_ext(product_action(a, b).act(g, p), p)
        product_action(a, b).act(g, p) = p
        stabilizer_contains_of_fixes(product_action(a, b), p, g)
        stabilizer(product_action(a, b), p).contains(g)
    }
}

/// The product-action stabilizer is the intersection of the coordinate stabilizers.
theorem product_action_stabilizer_eq_intersection[G: Group, X, Y](
    a: MulAction[G, X], b: MulAction[G, Y], p: Pair[X, Y], g: G) {
    stabilizer(product_action(a, b), p).contains(g) =
        stabilizer(a, p.first).intersection(stabilizer(b, p.second)).contains(g)
} by {
    if stabilizer(product_action(a, b), p).contains(g) {
        product_action_stabilizer_subset_first(a, b, p, g)
        product_action_stabilizer_subset_second(a, b, p, g)
        subgroup_intersection_contains_eq(stabilizer(a, p.first), stabilizer(b, p.second), g)
        stabilizer(a, p.first).intersection(stabilizer(b, p.second)).contains(g)
    }
    if stabilizer(a, p.first).intersection(stabilizer(b, p.second)).contains(g) {
        subgroup_intersection_contains_eq(stabilizer(a, p.first), stabilizer(b, p.second), g)
        stabilizer(a, p.first).contains(g)
        stabilizer(b, p.second).contains(g)
        product_action_stabilizer_contains_of_coordinates(a, b, p, g)
        stabilizer(product_action(a, b), p).contains(g)
    }
    stabilizer(product_action(a, b), p).contains(g) =
        stabilizer(a, p.first).intersection(stabilizer(b, p.second)).contains(g)
}

/// The product-action stabilizer is the intersection subgroup of the coordinate stabilizers.
theorem product_action_stabilizer_subgroup_eq_intersection[G: Group, X, Y](
    a: MulAction[G, X], b: MulAction[G, Y], p: Pair[X, Y]) {
    stabilizer(product_action(a, b), p) =
        stabilizer(a, p.first).intersection(stabilizer(b, p.second))
} by {
    forall(g: G) {
        product_action_stabilizer_eq_intersection(a, b, p, g)
        stabilizer(product_action(a, b), p).contains(g) =
            stabilizer(a, p.first).intersection(stabilizer(b, p.second)).contains(g)
    }
    subgroup_ext(stabilizer(product_action(a, b), p),
        stabilizer(a, p.first).intersection(stabilizer(b, p.second)))
}

/// The pointwise product of two maps.
define product_action_pair_map[X, Y, Z](f: X -> Y, h: X -> Z, x: X) -> Pair[Y, Z] {
    Pair.new(f(x), h(x))
}

/// The first coordinate of the pointwise product of two maps.
theorem product_action_pair_map_first[X, Y, Z](f: X -> Y, h: X -> Z, x: X) {
    product_action_pair_map(f, h, x).first = f(x)
}

/// The second coordinate of the pointwise product of two maps.
theorem product_action_pair_map_second[X, Y, Z](f: X -> Y, h: X -> Z, x: X) {
    product_action_pair_map(f, h, x).second = h(x)
}

/// The pointwise product of equivariant maps is equivariant for the product action.
theorem product_action_pair_map_is_equivariant[G: Group, X, Y, Z](
    a: MulAction[G, X],
    b: MulAction[G, Y],
    c: MulAction[G, Z],
    f: X -> Y,
    h: X -> Z
) {
    is_equivariant_map(a, b, f) and is_equivariant_map(a, c, h) implies
        is_equivariant_map(a, product_action(b, c), product_action_pair_map(f, h))
} by {
    if is_equivariant_map(a, b, f) and is_equivariant_map(a, c, h) {
        forall(g: G, x: X) {
            equivariant_map_apply(a, b, f, g, x)
            equivariant_map_apply(a, c, h, g, x)
            f(a.act(g, x)) = b.act(g, f(x))
            h(a.act(g, x)) = c.act(g, h(x))
            let lhs = product_action_pair_map(f, h, a.act(g, x))
            let rhs = product_action(b, c).act(g, product_action_pair_map(f, h, x))
            lhs.first = f(a.act(g, x))
            lhs.second = h(a.act(g, x))
            product_action_first(b, c, g, product_action_pair_map(f, h, x))
            product_action_second(b, c, g, product_action_pair_map(f, h, x))
            product_action_pair_map_first(f, h, x)
            product_action_pair_map_second(f, h, x)
            rhs.first = b.act(g, f(x))
            rhs.second = c.act(g, h(x))
            lhs.first = rhs.first
            lhs.second = rhs.second
            pair_ext(lhs, rhs)
            product_action_pair_map(f, h, a.act(g, x)) =
                product_action(b, c).act(g, product_action_pair_map(f, h, x))
        }
    }
}

/// The paired value of a shared witness belongs to the product orbit.
theorem product_action_orbit_contains_action_pair[G: Group, X, Y](
    a: MulAction[G, X], b: MulAction[G, Y], p: Pair[X, Y], g: G) {
    orbit(product_action(a, b), p).contains(Pair.new(a.act(g, p.first), b.act(g, p.second)))
} by {
    product_action_act(a, b, g, p)
    product_action(a, b).act(g, p) = Pair.new(a.act(g, p.first), b.act(g, p.second))
    orbit_contains_action(product_action(a, b), p, g)
    orbit(product_action(a, b), p).contains(product_action(a, b).act(g, p))
}

/// The paired value of a shared witness belongs to the product orbit from a constructed pair.
theorem product_action_orbit_contains_pair_new[G: Group, X, Y](
    a: MulAction[G, X], b: MulAction[G, Y], x: X, y: Y, g: G) {
    orbit(product_action(a, b), Pair.new(x, y)).contains(Pair.new(a.act(g, x), b.act(g, y)))
} by {
    let p = Pair.new(x, y)
    p.first = x
    p.second = y
    product_action_orbit_contains_action_pair(a, b, p, g)
    orbit(product_action(a, b), p).contains(Pair.new(a.act(g, p.first), b.act(g, p.second)))
    Pair.new(a.act(g, p.first), b.act(g, p.second)) = Pair.new(a.act(g, x), b.act(g, y))
}

/// Product-action orbit membership gives first-coordinate orbit membership.
theorem product_action_orbit_projects_first[G: Group, X, Y](
    a: MulAction[G, X], b: MulAction[G, Y], p: Pair[X, Y], q: Pair[X, Y]) {
    orbit(product_action(a, b), p).contains(q) implies orbit(a, p.first).contains(q.first)
} by {
    if orbit(product_action(a, b), p).contains(q) {
        product_action_orbit_projects(a, b, p, q)
    }
}

/// Product-action orbit membership gives second-coordinate orbit membership.
theorem product_action_orbit_projects_second[G: Group, X, Y](
    a: MulAction[G, X], b: MulAction[G, Y], p: Pair[X, Y], q: Pair[X, Y]) {
    orbit(product_action(a, b), p).contains(q) implies orbit(b, p.second).contains(q.second)
} by {
    if orbit(product_action(a, b), p).contains(q) {
        product_action_orbit_projects(a, b, p, q)
    }
}

/// Two action homomorphisms with a common source pair to an equivariant product map.
theorem product_action_pair_action_hom_maps_are_equivariant[G: Group, X, Y, Z](
    f: ActionHom[G, X, Y],
    h: ActionHom[G, X, Z]
) {
    f.src = h.src implies
        is_equivariant_map(f.src, product_action(f.dst, h.dst), product_action_pair_map(f.map, h.map))
} by {
    if f.src = h.src {
        is_equivariant_map(f.src, f.dst, f.map)
        is_equivariant_map(h.src, h.dst, h.map)
        is_equivariant_map(f.src, h.dst, h.map)
        product_action_pair_map_is_equivariant(f.src, f.dst, h.dst, f.map, h.map)
    }
}

/// The named first projection of a paired map is the first map.
theorem product_action_pair_map_first_projection[X, Y, Z](f: X -> Y, h: X -> Z, x: X) {
    product_first_projection[Y, Z](product_action_pair_map(f, h, x)) = f(x)
} by {
    product_first_projection[Y, Z](product_action_pair_map(f, h, x)) =
        product_action_pair_map(f, h, x).first
    product_action_pair_map_first(f, h, x)
}

/// The named second projection of a paired map is the second map.
theorem product_action_pair_map_second_projection[X, Y, Z](f: X -> Y, h: X -> Z, x: X) {
    product_second_projection[Y, Z](product_action_pair_map(f, h, x)) = h(x)
} by {
    product_second_projection[Y, Z](product_action_pair_map(f, h, x)) =
        product_action_pair_map(f, h, x).second
    product_action_pair_map_second(f, h, x)
}

/// A paired equivariant map sends orbit membership to product-action orbit membership.
theorem product_action_pair_map_maps_orbit_contains[G: Group, X, Y, Z](
    a: MulAction[G, X],
    b: MulAction[G, Y],
    c: MulAction[G, Z],
    f: X -> Y,
    h: X -> Z,
    x: X,
    y: X
) {
    is_equivariant_map(a, b, f) and is_equivariant_map(a, c, h) and orbit(a, x).contains(y) implies
        orbit(product_action(b, c), product_action_pair_map(f, h, x)).contains(
            product_action_pair_map(f, h, y))
} by {
    if is_equivariant_map(a, b, f) and is_equivariant_map(a, c, h) and orbit(a, x).contains(y) {
        product_action_pair_map_is_equivariant(a, b, c, f, h)
        is_equivariant_map(a, product_action(b, c), product_action_pair_map(f, h))
        equivariant_map_maps_orbit_contains(a, product_action(b, c), product_action_pair_map(f, h), x, y)
    }
}

/// A paired equivariant map sends stabilizer membership to product-action stabilizer membership.
theorem product_action_pair_map_maps_stabilizer_contains[G: Group, X, Y, Z](
    a: MulAction[G, X],
    b: MulAction[G, Y],
    c: MulAction[G, Z],
    f: X -> Y,
    h: X -> Z,
    x: X,
    g: G
) {
    is_equivariant_map(a, b, f) and is_equivariant_map(a, c, h) and stabilizer(a, x).contains(g) implies
        stabilizer(product_action(b, c), product_action_pair_map(f, h, x)).contains(g)
} by {
    if is_equivariant_map(a, b, f) and is_equivariant_map(a, c, h) and stabilizer(a, x).contains(g) {
        product_action_pair_map_is_equivariant(a, b, c, f, h)
        is_equivariant_map(a, product_action(b, c), product_action_pair_map(f, h))
        equivariant_map_maps_stabilizer_contains(a, product_action(b, c), product_action_pair_map(f, h), x, g)
    }
}

/// Product-action transitivity implies transitivity of the first coordinate action when the second coordinate has a base point.
theorem product_action_transitive_implies_first_transitive[G: Group, X, Y](
    a: MulAction[G, X], b: MulAction[G, Y], y0: Y) {
    is_transitive_action(product_action(a, b)) implies is_transitive_action(a)
} by {
    if is_transitive_action(product_action(a, b)) {
        forall(x1: X, x2: X) {
            let p = Pair.new(x1, y0)
            let q = Pair.new(x2, y0)
            transitive_action_orbit_contains(product_action(a, b), p, q)
            orbit(product_action(a, b), p).contains(q)
            product_action_orbit_projects_first(a, b, p, q)
            orbit(a, p.first).contains(q.first)
            p.first = x1
            q.first = x2
            orbit(a, x1).contains(x2)
        }
        is_transitive_action(a)
    }
}

/// Product-action transitivity implies transitivity of the second coordinate action when the first coordinate has a base point.
theorem product_action_transitive_implies_second_transitive[G: Group, X, Y](
    a: MulAction[G, X], b: MulAction[G, Y], x0: X) {
    is_transitive_action(product_action(a, b)) implies is_transitive_action(b)
} by {
    if is_transitive_action(product_action(a, b)) {
        forall(y1: Y, y2: Y) {
            let p = Pair.new(x0, y1)
            let q = Pair.new(x0, y2)
            transitive_action_orbit_contains(product_action(a, b), p, q)
            orbit(product_action(a, b), p).contains(q)
            product_action_orbit_projects_second(a, b, p, q)
            orbit(b, p.second).contains(q.second)
            p.second = y1
            q.second = y2
            orbit(b, y1).contains(y2)
        }
        is_transitive_action(b)
    }
}

/// If the first coordinate action is faithful, then the product action is faithful.
theorem product_action_faithful_of_first[G: Group, X, Y](
    a: MulAction[G, X], b: MulAction[G, Y], y0: Y) {
    is_faithful_action(a) implies is_faithful_action(product_action(a, b))
} by {
    if is_faithful_action(a) {
        forall(g: G, h: G) {
            if forall(p: Pair[X, Y]) { product_action(a, b).act(g, p) = product_action(a, b).act(h, p) } {
                forall(x: X) {
                    let p = Pair.new(x, y0)
                    product_action_first(a, b, g, p)
                    product_action_first(a, b, h, p)
                    product_action(a, b).act(g, p) = product_action(a, b).act(h, p)
                    product_action(a, b).act(g, p).first = product_action(a, b).act(h, p).first
                    a.act(g, p.first) = a.act(h, p.first)
                    p.first = x
                    a.act(g, x) = a.act(h, x)
                }
                faithful_action_eq_of_pointwise_eq(a, g, h)
                g = h
            }
        }
        is_faithful_action(product_action(a, b))
    }
}

/// If the second coordinate action is faithful, then the product action is faithful.
theorem product_action_faithful_of_second[G: Group, X, Y](
    a: MulAction[G, X], b: MulAction[G, Y], x0: X) {
    is_faithful_action(b) implies is_faithful_action(product_action(a, b))
} by {
    if is_faithful_action(b) {
        forall(g: G, h: G) {
            if forall(p: Pair[X, Y]) { product_action(a, b).act(g, p) = product_action(a, b).act(h, p) } {
                forall(y: Y) {
                    let p = Pair.new(x0, y)
                    product_action_second(a, b, g, p)
                    product_action_second(a, b, h, p)
                    product_action(a, b).act(g, p) = product_action(a, b).act(h, p)
                    product_action(a, b).act(g, p).second = product_action(a, b).act(h, p).second
                    b.act(g, p.second) = b.act(h, p.second)
                    p.second = y
                    b.act(g, y) = b.act(h, y)
                }
                faithful_action_eq_of_pointwise_eq(b, g, h)
                g = h
            }
        }
        is_faithful_action(product_action(a, b))
    }
}
