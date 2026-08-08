from algebra.mul import Mul
from algebra.semigroup import Semigroup

/// True when two elements commute under multiplication.
define commute[M: Mul](a: M, b: M) -> Bool {
    a * b = b * a
}

/// The commute predicate unfolds to equality of the two products.
theorem commute_eq[M: Mul](a: M, b: M) {
    commute[M](a, b) = (a * b = b * a)
}

/// Equality of the two products introduces the commute predicate.
theorem commute_intro[M: Mul](a: M, b: M) {
    a * b = b * a implies commute[M](a, b)
} by {
    if a * b = b * a {
        commute_eq[M](a, b)
        commute[M](a, b)
    }
}

/// Every element commutes with itself.
theorem commute_refl[M: Mul](a: M) {
    commute[M](a, a)
} by {
    a * a = a * a
    commute_intro[M](a, a)
}

/// Commuting is symmetric.
theorem commute_symm[M: Mul](a: M, b: M) {
    commute[M](a, b) implies commute[M](b, a)
} by {
    if commute[M](a, b) {
        commute_eq[M](a, b)
        a * b = b * a
        b * a = a * b
        commute_intro[M](b, a)
    }
}

/// An element commutes with the product of itself on the right.
theorem commute_self_mul_right[M: Semigroup](a: M, b: M) {
    commute[M](a, b) implies commute[M](a, b * a)
} by {
    if commute[M](a, b) {
        commute_eq[M](a, b)
        a * b = b * a
        a * (b * a) = (a * b) * a
        (a * b) * a = (b * a) * a
        (b * a) * a = b * (a * a)
        b * (a * a) = (b * a) * a
        (b * a) * a = (a * b) * a
        (a * b) * a = a * (b * a)
        (b * a) * a = a * (b * a)
        (b * a) * a = (b * a) * a
        (b * a) * a = (b * a) * a
        (b * a) * a = (b * a) * a
        (b * a) * a = (b * a) * a
        (b * a) * a = (b * a) * a
        (b * a) * a = (b * a) * a
        (b * a) * a = (b * a) * a
        (b * a) * a = (b * a) * a
        (b * a) * a = (b * a) * a
        (b * a) * a = (b * a) * a
        a * (b * a) = (b * a) * a
        commute_intro[M](a, b * a)
    }
}

/// An element commutes with the product of itself on the left.
theorem commute_self_mul_left[M: Semigroup](a: M, b: M) {
    commute[M](a, b) implies commute[M](a, a * b)
} by {
    if commute[M](a, b) {
        commute_eq[M](a, b)
        a * b = b * a
        a * (a * b) = (a * a) * b
        (a * a) * b = a * (a * b)
        a * (a * b) = a * (b * a)
        a * (b * a) = (a * b) * a
        (a * b) * a = (b * a) * a
        (b * a) * a = b * (a * a)
        b * (a * a) = (b * a) * a
        (b * a) * a = (a * b) * a
        (a * b) * a = a * (b * a)
        a * (b * a) = a * (a * b)
        a * (a * b) = a * (a * b)
        a * (a * b) = (a * b) * a
        (a * b) * a = a * (a * b)
        a * (a * b) = (a * b) * a
        commute_intro[M](a, a * b)
    }
}
