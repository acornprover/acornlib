from comm_ring import CommRing
from algebra.field.field import Field, pow_not_zero
from nat import Nat, alt_induction
from algebra.units import is_monoid_unit, is_monoid_unit_of_inverse,
    monoid_unit_mul_is_monoid_unit, monoid_unit_pow_is_monoid_unit,
    monoid_unit_mul_nonzero_left, monoid_unit_mul_nonzero_right,
    monoid_unit_mul_cancel_left, monoid_unit_mul_cancel_right,
    monoid_unit_mul_zero_iff_left, monoid_unit_mul_zero_iff_right,
    field_nonzero_is_monoid_unit
from algebra.ring.ideal import is_ideal, is_ideal_absorb, ideal_eq_unit_of_contains_one, unit_ideal,
    ideal_proper_constraint, ideal_subset, principal_ideal, principal_ideal_is_ideal,
    principal_ideal_contains_generator

attributes R: CommRing {
    /// True if `a` is a unit in the commutative ring `R`: there exists `b` with `a * b = R.1`.
    define is_unit(self) -> Bool {
        exists(b: R) { self * b = R.1 }
    }

    /// True if `a` is not a unit in the commutative ring `R`.
    define is_non_unit(self) -> Bool {
        not self.is_unit
    }
}

/// The multiplicative identity is a unit.
theorem is_unit_one[R: CommRing] {
    R.1.is_unit
} by {
}

/// The commutative-ring unit predicate agrees with the monoid-unit predicate.
theorem is_unit_eq_is_monoid_unit[R: CommRing](a: R) {
    a.is_unit = is_monoid_unit(a)
} by {
    if a.is_unit {
        let b: R satisfy { a * b = R.1 }
        is_monoid_unit_of_inverse(a, b)
        is_monoid_unit(a)
    }
    if is_monoid_unit(a) {
        let b: R satisfy {
            a * b = R.1 and b * a = R.1
        }
        a.is_unit
    }
}

/// A monoid unit is a unit in a commutative ring.
theorem is_unit_of_monoid_unit[R: CommRing](a: R) {
    is_monoid_unit(a) implies a.is_unit
} by {
    is_unit_eq_is_monoid_unit(a)
}

/// A commutative-ring unit is a monoid unit.
theorem is_monoid_unit_of_is_unit[R: CommRing](a: R) {
    a.is_unit implies is_monoid_unit(a)
} by {
    is_unit_eq_is_monoid_unit(a)
}

/// The monoid-unit and commutative-ring unit predicates are equivalent in implication form.
theorem is_unit_iff_is_monoid_unit[R: CommRing](a: R) {
    a.is_unit = is_monoid_unit(a)
} by {
    is_unit_eq_is_monoid_unit(a)
}

/// `a.is_unit` unfolds to the existence of a multiplicative right-inverse.
theorem is_unit_exists[R: CommRing](a: R) {
    a.is_unit implies exists(b: R) { a * b = R.1 }
}

/// A witness for `a * b = R.1` exhibits `a` as a unit.
theorem is_unit_witness[R: CommRing](a: R, b: R) {
    a * b = R.1 implies a.is_unit
} by {
    if a * b = R.1 {
        exists(c: R) { a * c = R.1 }
    }
}

/// In a commutative ring, units are symmetric in the witnessing equation.
theorem is_unit_symm[R: CommRing](a: R, b: R) {
    a * b = R.1 implies b * a = R.1
} by {
    if a * b = R.1 {
        b * a = a * b
    }
}

/// The product of two units is a unit.
theorem is_unit_mul[R: CommRing](a: R, b: R) {
    a.is_unit and b.is_unit implies (a * b).is_unit
} by {
    if a.is_unit and b.is_unit {
        is_monoid_unit_of_is_unit(a)
        is_monoid_unit_of_is_unit(b)
        monoid_unit_mul_is_monoid_unit(a, b)
        is_unit_eq_is_monoid_unit(a * b)
        (a * b).is_unit
    }
}

/// A commutative-ring unit stays a unit under natural powers.
theorem is_unit_pow_via_monoid_unit[R: CommRing](a: R, n: Nat) {
    a.is_unit implies a.pow(n).is_unit
} by {
    if a.is_unit {
        is_monoid_unit_of_is_unit(a)
        monoid_unit_pow_is_monoid_unit(a, n)
        is_unit_eq_is_monoid_unit(a.pow(n))
        a.pow(n).is_unit
    }
}

/// Multiplying a unit by a unit power gives another unit power.
theorem is_unit_pow_suc[R: CommRing](a: R, k: Nat) {
    a.is_unit and a.pow(k).is_unit implies a.pow(k.suc).is_unit
} by {
    if a.is_unit and a.pow(k).is_unit {
        is_unit_mul(a, a.pow(k))
        a.pow(k.suc).is_unit
    }
}

/// Every natural power of a unit is a unit.
theorem is_unit_pow[R: CommRing](a: R, n: Nat) {
    a.is_unit implies a.pow(n).is_unit
} by {
    define p(k: Nat) -> Bool {
        a.is_unit implies a.pow(k).is_unit
    }

    if a.is_unit {
        is_unit_one[R]
        a.pow(Nat.0).is_unit
    }
    p(Nat.0)

    forall(k: Nat) {
        if p(k) {
            if a.is_unit {
                is_unit_pow_suc(a, k)
                a.pow(k.suc).is_unit
            }
            p(k.suc)
        }
    }
    p(Nat.0) and forall(k: Nat) {
        p(k) implies p(k.suc)
    }
    alt_induction(p)
    forall(k: Nat) {
        p(k)
    }

    if a.is_unit {
        a.pow(n).is_unit
    }
}

/// The zero element is a non-unit unless `R.0 = R.1` (a trivial ring).
theorem zero_non_unit_or_trivial[R: CommRing] {
    R.0.is_non_unit or R.0 = R.1
} by {
    if R.0 != R.1 {
        if R.0.is_unit {
            let b: R satisfy { R.0 * b = R.1 }
            false
        }
        R.0.is_non_unit
    }
}

/// Multiplication by a unit on the left is cancellative in a commutative ring.
theorem is_unit_mul_cancel_left[R: CommRing](u: R, a: R, b: R) {
    u.is_unit and u * a = u * b implies a = b
} by {
    if u.is_unit and u * a = u * b {
        is_monoid_unit_of_is_unit(u)
        monoid_unit_mul_cancel_left(u, a, b)
        a = b
    }
}

/// Multiplication by a unit on the right is cancellative in a commutative ring.
theorem is_unit_mul_cancel_right[R: CommRing](u: R, a: R, b: R) {
    u.is_unit and a * u = b * u implies a = b
} by {
    if u.is_unit and a * u = b * u {
        is_monoid_unit_of_is_unit(u)
        monoid_unit_mul_cancel_right(u, a, b)
        a = b
    }
}

/// A product by a unit on the left is zero exactly when the other factor is zero.
theorem is_unit_mul_zero_iff_left[R: CommRing](u: R, a: R) {
    u.is_unit implies ((u * a = R.0) = (a = R.0))
} by {
    if u.is_unit {
        is_monoid_unit_of_is_unit(u)
        monoid_unit_mul_zero_iff_left(u, a)
        (u * a = R.0) = (a = R.0)
    }
}

/// A product by a unit on the right is zero exactly when the other factor is zero.
theorem is_unit_mul_zero_iff_right[R: CommRing](u: R, a: R) {
    u.is_unit implies ((a * u = R.0) = (a = R.0))
} by {
    if u.is_unit {
        is_monoid_unit_of_is_unit(u)
        monoid_unit_mul_zero_iff_right(u, a)
        (a * u = R.0) = (a = R.0)
    }
}

/// A unit cannot annihilate a nonzero element on the left.
theorem is_unit_mul_nonzero_left[R: CommRing](u: R, a: R) {
    u.is_unit and a != R.0 implies u * a != R.0
} by {
    if u.is_unit and a != R.0 {
        is_monoid_unit_of_is_unit(u)
        monoid_unit_mul_nonzero_left(u, a)
        u * a != R.0
    }
}

/// A unit cannot annihilate a nonzero element on the right.
theorem is_unit_mul_nonzero_right[R: CommRing](u: R, a: R) {
    u.is_unit and a != R.0 implies a * u != R.0
} by {
    if u.is_unit and a != R.0 {
        is_monoid_unit_of_is_unit(u)
        monoid_unit_mul_nonzero_right(u, a)
        a * u != R.0
    }
}

/// Left multiplication by a unit preserves and reflects equality.
theorem is_unit_mul_eq_iff_left[R: CommRing](u: R, a: R, b: R) {
    u.is_unit implies ((u * a = u * b) = (a = b))
} by {
    if u.is_unit {
        if u * a = u * b {
            is_unit_mul_cancel_left(u, a, b)
            a = b
        }
        if a = b {
            u * a = u * b
        }
    }
}

/// Right multiplication by a unit preserves and reflects equality.
theorem is_unit_mul_eq_iff_right[R: CommRing](u: R, a: R, b: R) {
    u.is_unit implies ((a * u = b * u) = (a = b))
} by {
    if u.is_unit {
        if a * u = b * u {
            is_unit_mul_cancel_right(u, a, b)
            a = b
        }
        if a = b {
            a * u = b * u
        }
    }
}

/// Non-units are contrapositive-stable under multiplication by a unit on the left.
theorem is_non_unit_of_mul_left[R: CommRing](u: R, a: R) {
    u.is_unit and (u * a).is_non_unit implies a.is_non_unit
} by {
    if u.is_unit and (u * a).is_non_unit {
        if a.is_unit {
            is_unit_mul(u, a)
            false
        }
        a.is_non_unit
    }
}

/// Non-units are contrapositive-stable under multiplication by a unit on the right.
theorem is_non_unit_of_mul_right[R: CommRing](u: R, a: R) {
    u.is_unit and (a * u).is_non_unit implies a.is_non_unit
} by {
    if u.is_unit and (a * u).is_non_unit {
        if a.is_unit {
            is_unit_mul(a, u)
            false
        }
        a.is_non_unit
    }
}

/// In a field, every nonzero element is a unit.
theorem field_nonzero_is_unit[F: Field](a: F) {
    a != F.0 implies a.is_unit
} by {
    if a != F.0 {
        field_nonzero_is_monoid_unit(a)
        is_unit_of_monoid_unit(a)
        a.is_unit
    }
}

/// Every natural power of a nonzero field element is a unit.
theorem field_nonzero_pow_is_unit[F: Field](a: F, n: Nat) {
    a != F.0 implies a.pow(n).is_unit
} by {
    if a != F.0 {
        pow_not_zero(a, n)
        field_nonzero_is_unit(a.pow(n))
        a.pow(n).is_unit
    }
}

/// A unit of a field is nonzero.
theorem field_unit_is_nonzero[F: Field](a: F) {
    a.is_unit implies a != F.0
} by {
    if a.is_unit {
        let b: F satisfy { a * b = F.1 }
        if a = F.0 {
            F.1 = F.0
        }
    }
}

/// In a field, being a unit is the same as being nonzero.
theorem field_is_unit_iff_nonzero[F: Field](a: F) {
    a.is_unit = (a != F.0)
} by {
    if a.is_unit {
        field_unit_is_nonzero(a)
        a != F.0
    }
    if a != F.0 {
        field_nonzero_is_unit(a)
        a.is_unit
    }
}

/// The zero element of a field is a non-unit.
theorem field_zero_is_non_unit[F: Field] {
    F.0.is_non_unit
} by {
    field_is_unit_iff_nonzero(F.0)
}

/// The set of non-units of a commutative ring, as a membership predicate.
define non_units[R: CommRing](a: R) -> Bool {
    a.is_non_unit
}

/// Membership in `non_units` coincides with `is_non_unit`.
theorem non_units_mem[R: CommRing](a: R) {
    non_units[R](a) = a.is_non_unit
}

/// The multiplicative identity is not a non-unit.
theorem non_units_not_one[R: CommRing] {
    not non_units[R](R.1)
} by {
    is_unit_one[R]
    not R.1.is_non_unit
}

/// An ideal containing a unit equals the whole ring.
theorem ideal_with_unit_eq_unit_ideal[R: CommRing](contains: R -> Bool, u: R) {
    is_ideal(contains) and contains(u) and u.is_unit implies contains = unit_ideal[R]
} by {
    if is_ideal(contains) and contains(u) and u.is_unit {
        is_unit_exists(u)
        let v: R satisfy { u * v = R.1 }
        is_ideal_absorb(contains, v, u)
        ideal_eq_unit_of_contains_one(contains)
        contains = unit_ideal[R]
    }
}

/// The principal ideal generated by a unit is the whole ring.
theorem principal_ideal_eq_unit_of_unit[R: CommRing](a: R) {
    a.is_unit implies principal_ideal[R](a) = unit_ideal[R]
} by {
    if a.is_unit {
        principal_ideal_is_ideal(a)
        principal_ideal_contains_generator(a)
        ideal_with_unit_eq_unit_ideal(principal_ideal[R](a), a)
        principal_ideal[R](a) = unit_ideal[R]
    }
}

/// If the principal ideal generated by an element is the whole ring, then the element is a unit.
theorem unit_of_principal_ideal_eq_unit[R: CommRing](a: R) {
    principal_ideal[R](a) = unit_ideal[R] implies a.is_unit
} by {
    if principal_ideal[R](a) = unit_ideal[R] {
        principal_ideal(a, R.1)
        let b: R satisfy {
            R.1 = b * a
        }
        is_unit_witness(a, b)
        a.is_unit
    }
}

/// A proper ideal contains no units: every element of a proper ideal is a non-unit.
theorem proper_ideal_no_units[R: CommRing](contains: R -> Bool, a: R) {
    is_ideal(contains) and ideal_proper_constraint(contains) and contains(a) implies a.is_non_unit
} by {
    if is_ideal(contains) and ideal_proper_constraint(contains) and contains(a) {
        if a.is_unit {
            ideal_with_unit_eq_unit_ideal(contains, a)
            contains(R.1)
            false
        }
        a.is_non_unit
    }
}

/// Every proper ideal is contained in the set of non-units.
theorem proper_ideal_subset_non_units[R: CommRing](contains: R -> Bool) {
    is_ideal(contains) and ideal_proper_constraint(contains) implies ideal_subset(contains, non_units[R])
} by {
    if is_ideal(contains) and ideal_proper_constraint(contains) {
        forall(x: R) {
            if contains(x) {
                proper_ideal_no_units(contains, x)
                non_units[R](x)
            }
        }
    }
}

/// In a field, a non-unit is exactly the zero element.
theorem field_non_unit_iff_zero[F: Field](a: F) {
    a.is_non_unit = (a = F.0)
} by {
    if a = F.0 {
        if a.is_unit {
            let b: F satisfy { a * b = F.1 }
            false
        }
        a.is_non_unit
    }
    if a.is_non_unit {
        if a != F.0 {
            field_nonzero_is_unit(a)
            false
        }
        a = F.0
    }
}
