from algebra.semigroup import Semigroup
from data.basic.set import Set, subset_contains
from algebra.subsemigroup import Subsemigroup, subsemigroup_subset, subsemigroup_subset_as_set_eq,
    subsemigroup_subset_refl, subsemigroup_subset_trans, subsemigroup_subset_antisymm,
    subsemigroup_intersection_subset_left_relation, subsemigroup_intersection_subset_right_relation,
    subsemigroup_subset_intersection_of_subset_left_right, subsemigroup_subset_intersection_iff,
    subsemigroup_closure, subsemigroup_closure_mono, subsemigroup_closure_le_iff_set_subset,
    set_subset_subsemigroup_as_set_eq, subsemigroup_sup, subsemigroup_subset_sup_left,
    subsemigroup_subset_sup_right, subsemigroup_sup_subset_of_subset_left_right

/// Subsemigroup inclusion is exactly elementwise implication of membership predicates.
theorem subsemigroup_subset_contains_eq[S: Semigroup](a: Subsemigroup[S], b: Subsemigroup[S]) {
    subsemigroup_subset(a, b) = forall(x: S) {
        a.contains(x) implies b.contains(x)
    }
} by {
    subsemigroup_subset(a, b) = forall(x: S) {
        a.contains(x) implies b.contains(x)
    }
}

/// A subsemigroup inclusion transports membership from the smaller subsemigroup to the larger one.
theorem subsemigroup_subset_contains[S: Semigroup](a: Subsemigroup[S], b: Subsemigroup[S], x: S) {
    subsemigroup_subset(a, b) and a.contains(x) implies b.contains(x)
} by {
    if subsemigroup_subset(a, b) and a.contains(x) {
        subsemigroup_subset_contains_eq(a, b)
        a.contains(x) implies b.contains(x)
        b.contains(x)
    }
}

/// Elementwise membership implication gives subsemigroup inclusion.
theorem subsemigroup_subset_of_contains[S: Semigroup](a: Subsemigroup[S], b: Subsemigroup[S]) {
    (forall(x: S) { a.contains(x) implies b.contains(x) }) implies subsemigroup_subset(a, b)
} by {
    if forall(x: S) { a.contains(x) implies b.contains(x) } {
        subsemigroup_subset_contains_eq(a, b)
        subsemigroup_subset(a, b)
    }
}

/// Subsemigroup inclusion is reflexive.
theorem subsemigroup_order_subset_refl[S: Semigroup](a: Subsemigroup[S]) {
    subsemigroup_subset(a, a)
} by {
    subsemigroup_subset_refl(a)
}

/// Subsemigroup inclusion is transitive.
theorem subsemigroup_order_subset_trans[S: Semigroup](
    a: Subsemigroup[S],
    b: Subsemigroup[S],
    c: Subsemigroup[S]
) {
    subsemigroup_subset(a, b) and subsemigroup_subset(b, c) implies subsemigroup_subset(a, c)
} by {
    if subsemigroup_subset(a, b) and subsemigroup_subset(b, c) {
        subsemigroup_subset_trans(a, b, c)
    }
}

/// Mutual subsemigroup inclusion forces equality.
theorem subsemigroup_order_subset_antisymm[S: Semigroup](a: Subsemigroup[S], b: Subsemigroup[S]) {
    subsemigroup_subset(a, b) and subsemigroup_subset(b, a) implies a = b
} by {
    if subsemigroup_subset(a, b) and subsemigroup_subset(b, a) {
        subsemigroup_subset_antisymm(a, b)
    }
}

/// Subsemigroup inclusion is exactly inclusion of the underlying sets.
theorem subsemigroup_subset_as_set_iff[S: Semigroup](a: Subsemigroup[S], b: Subsemigroup[S]) {
    subsemigroup_subset(a, b) = a.as_set.subset(b.as_set)
} by {
    subsemigroup_subset_as_set_eq(a, b)
}

/// Underlying-set inclusion transports subsemigroup membership.
theorem subsemigroup_as_set_subset_contains[S: Semigroup](a: Subsemigroup[S], b: Subsemigroup[S], x: S) {
    a.as_set.subset(b.as_set) and a.contains(x) implies b.contains(x)
} by {
    if a.as_set.subset(b.as_set) and a.contains(x) {
        a.as_set.contains(x)
        subset_contains(a.as_set, b.as_set, x)
        b.as_set.contains(x)
        subsemigroup_subset_as_set_eq(a, b)
        subsemigroup_subset(a, b)
        subsemigroup_subset_contains(a, b, x)
    }
}

/// The intersection is the greatest lower bound for subsemigroup inclusion.
theorem subsemigroup_intersection_greatest_lower_bound[S: Semigroup](
    c: Subsemigroup[S],
    a: Subsemigroup[S],
    b: Subsemigroup[S]
) {
    subsemigroup_subset(c, a.intersection(b)) = (subsemigroup_subset(c, a) and subsemigroup_subset(c, b))
} by {
    subsemigroup_subset_intersection_iff(c, a, b)
}

/// A subsemigroup contained in an intersection is contained in the left factor.
theorem subsemigroup_subset_left_of_subset_intersection[S: Semigroup](
    c: Subsemigroup[S],
    a: Subsemigroup[S],
    b: Subsemigroup[S]
) {
    subsemigroup_subset(c, a.intersection(b)) implies subsemigroup_subset(c, a)
} by {
    if subsemigroup_subset(c, a.intersection(b)) {
        subsemigroup_intersection_subset_left_relation(a, b)
        subsemigroup_subset_trans(c, a.intersection(b), a)
        subsemigroup_subset(c, a)
    }
}

/// A subsemigroup contained in an intersection is contained in the right factor.
theorem subsemigroup_subset_right_of_subset_intersection[S: Semigroup](
    c: Subsemigroup[S],
    a: Subsemigroup[S],
    b: Subsemigroup[S]
) {
    subsemigroup_subset(c, a.intersection(b)) implies subsemigroup_subset(c, b)
} by {
    if subsemigroup_subset(c, a.intersection(b)) {
        subsemigroup_intersection_subset_right_relation(a, b)
        subsemigroup_subset_trans(c, a.intersection(b), b)
        subsemigroup_subset(c, b)
    }
}

/// A common subsemigroup of two subsemigroups is contained in their intersection.
theorem subsemigroup_subset_intersection_of_subset_both[S: Semigroup](
    c: Subsemigroup[S],
    a: Subsemigroup[S],
    b: Subsemigroup[S]
) {
    subsemigroup_subset(c, a) and subsemigroup_subset(c, b) implies subsemigroup_subset(c, a.intersection(b))
} by {
    if subsemigroup_subset(c, a) and subsemigroup_subset(c, b) {
        subsemigroup_subset_intersection_of_subset_left_right(c, a, b)
    }
}

/// Subsemigroup closure is monotone with respect to inclusion of generating sets.
theorem subsemigroup_closure_monotone[S: Semigroup](a: Set[S], b: Set[S]) {
    a.subset(b) implies subsemigroup_subset(subsemigroup_closure(a), subsemigroup_closure(b))
} by {
    if a.subset(b) {
        subsemigroup_closure_mono(a, b)
    }
}

/// The underlying set of subsemigroup closure is monotone with respect to inclusion of generating sets.
theorem subsemigroup_closure_as_set_monotone[S: Semigroup](a: Set[S], b: Set[S]) {
    a.subset(b) implies subsemigroup_closure(a).as_set.subset(subsemigroup_closure(b).as_set)
} by {
    if a.subset(b) {
        subsemigroup_closure_monotone(a, b)
        subsemigroup_subset_as_set_eq(subsemigroup_closure(a), subsemigroup_closure(b))
        subsemigroup_closure(a).as_set.subset(subsemigroup_closure(b).as_set)
    }
}

/// The closure of a set is contained in a subsemigroup exactly when the set is contained in its underlying set.
theorem subsemigroup_closure_subset_iff_as_set_subset[S: Semigroup](a: Set[S], s: Subsemigroup[S]) {
    subsemigroup_subset(subsemigroup_closure(a), s) = a.subset(s.as_set)
} by {
    subsemigroup_closure_le_iff_set_subset(a, s)
    set_subset_subsemigroup_as_set_eq(a, s)
}

/// Mutual inclusion of subsemigroups is equivalent to equality.
theorem subsemigroup_subset_antisymm_iff_eq[S: Semigroup](a: Subsemigroup[S], b: Subsemigroup[S]) {
    (subsemigroup_subset(a, b) and subsemigroup_subset(b, a)) = (a = b)
} by {
    if subsemigroup_subset(a, b) and subsemigroup_subset(b, a) {
        subsemigroup_subset_antisymm(a, b)
        a = b
    }
    if a = b {
        subsemigroup_subset_refl(a)
        subsemigroup_subset(a, b)
        subsemigroup_subset(b, a)
    }
    (subsemigroup_subset(a, b) and subsemigroup_subset(b, a)) = (a = b)
}

/// Intersections of subsemigroups are monotone in the left argument.
theorem subsemigroup_intersection_mono_left[S: Semigroup](
    a: Subsemigroup[S],
    b: Subsemigroup[S],
    c: Subsemigroup[S]
) {
    subsemigroup_subset(a, b) implies subsemigroup_subset(a.intersection(c), b.intersection(c))
} by {
    if subsemigroup_subset(a, b) {
        subsemigroup_intersection_subset_left_relation(a, c)
        subsemigroup_subset_trans(a.intersection(c), a, b)
        subsemigroup_subset(a.intersection(c), b)
        subsemigroup_intersection_subset_right_relation(a, c)
        subsemigroup_subset(a.intersection(c), c)
        subsemigroup_subset_intersection_of_subset_left_right(a.intersection(c), b, c)
        subsemigroup_subset(a.intersection(c), b.intersection(c))
    }
}

/// Intersections of subsemigroups are monotone in the right argument.
theorem subsemigroup_intersection_mono_right[S: Semigroup](
    a: Subsemigroup[S],
    b: Subsemigroup[S],
    c: Subsemigroup[S]
) {
    subsemigroup_subset(a, b) implies subsemigroup_subset(c.intersection(a), c.intersection(b))
} by {
    if subsemigroup_subset(a, b) {
        subsemigroup_intersection_subset_left_relation(c, a)
        subsemigroup_subset(c.intersection(a), c)
        subsemigroup_intersection_subset_right_relation(c, a)
        subsemigroup_subset_trans(c.intersection(a), a, b)
        subsemigroup_subset(c.intersection(a), b)
        subsemigroup_subset_intersection_of_subset_left_right(c.intersection(a), c, b)
        subsemigroup_subset(c.intersection(a), c.intersection(b))
    }
}

/// Intersections of subsemigroups are monotone in both arguments.
theorem subsemigroup_intersection_mono[S: Semigroup](
    a: Subsemigroup[S],
    b: Subsemigroup[S],
    c: Subsemigroup[S],
    d: Subsemigroup[S]
) {
    subsemigroup_subset(a, b) and subsemigroup_subset(c, d) implies
        subsemigroup_subset(a.intersection(c), b.intersection(d))
} by {
    if subsemigroup_subset(a, b) and subsemigroup_subset(c, d) {
        subsemigroup_intersection_mono_left(a, b, c)
        subsemigroup_subset(a.intersection(c), b.intersection(c))
        subsemigroup_intersection_mono_right(c, d, b)
        subsemigroup_subset(b.intersection(c), b.intersection(d))
        subsemigroup_subset_trans(a.intersection(c), b.intersection(c), b.intersection(d))
        subsemigroup_subset(a.intersection(c), b.intersection(d))
    }
}

/// Joins of subsemigroups are monotone in the left argument.
theorem subsemigroup_sup_mono_left[S: Semigroup](
    a: Subsemigroup[S],
    b: Subsemigroup[S],
    c: Subsemigroup[S]
) {
    subsemigroup_subset(a, b) implies subsemigroup_subset(subsemigroup_sup(a, c), subsemigroup_sup(b, c))
} by {
    if subsemigroup_subset(a, b) {
        subsemigroup_subset_sup_left(b, c)
        subsemigroup_subset_trans(a, b, subsemigroup_sup(b, c))
        subsemigroup_subset(a, subsemigroup_sup(b, c))
        subsemigroup_subset_sup_right(b, c)
        subsemigroup_subset(c, subsemigroup_sup(b, c))
        subsemigroup_sup_subset_of_subset_left_right(a, c, subsemigroup_sup(b, c))
        subsemigroup_subset(subsemigroup_sup(a, c), subsemigroup_sup(b, c))
    }
}

/// Joins of subsemigroups are monotone in the right argument.
theorem subsemigroup_sup_mono_right[S: Semigroup](
    a: Subsemigroup[S],
    b: Subsemigroup[S],
    c: Subsemigroup[S]
) {
    subsemigroup_subset(a, b) implies subsemigroup_subset(subsemigroup_sup(c, a), subsemigroup_sup(c, b))
} by {
    if subsemigroup_subset(a, b) {
        subsemigroup_subset_sup_left(c, b)
        subsemigroup_subset(c, subsemigroup_sup(c, b))
        subsemigroup_subset_sup_right(c, b)
        subsemigroup_subset_trans(a, b, subsemigroup_sup(c, b))
        subsemigroup_subset(a, subsemigroup_sup(c, b))
        subsemigroup_sup_subset_of_subset_left_right(c, a, subsemigroup_sup(c, b))
        subsemigroup_subset(subsemigroup_sup(c, a), subsemigroup_sup(c, b))
    }
}

/// Joins of subsemigroups are monotone in both arguments.
theorem subsemigroup_sup_mono[S: Semigroup](
    a: Subsemigroup[S],
    b: Subsemigroup[S],
    c: Subsemigroup[S],
    d: Subsemigroup[S]
) {
    subsemigroup_subset(a, b) and subsemigroup_subset(c, d) implies
        subsemigroup_subset(subsemigroup_sup(a, c), subsemigroup_sup(b, d))
} by {
    if subsemigroup_subset(a, b) and subsemigroup_subset(c, d) {
        subsemigroup_subset_sup_left(b, d)
        subsemigroup_subset_trans(a, b, subsemigroup_sup(b, d))
        subsemigroup_subset(a, subsemigroup_sup(b, d))
        subsemigroup_subset_sup_right(b, d)
        subsemigroup_subset_trans(c, d, subsemigroup_sup(b, d))
        subsemigroup_subset(c, subsemigroup_sup(b, d))
        subsemigroup_sup_subset_of_subset_left_right(a, c, subsemigroup_sup(b, d))
        subsemigroup_subset(subsemigroup_sup(a, c), subsemigroup_sup(b, d))
    }
}

/// Mutual containment in generated subsemigroups identifies two subsemigroup closures.
theorem subsemigroup_closure_eq_of_mutual_as_set_subset[S: Semigroup](a: Set[S], b: Set[S]) {
    a.subset(subsemigroup_closure(b).as_set) and b.subset(subsemigroup_closure(a).as_set) implies
        subsemigroup_closure(a) = subsemigroup_closure(b)
} by {
    if a.subset(subsemigroup_closure(b).as_set) and b.subset(subsemigroup_closure(a).as_set) {
        subsemigroup_closure_subset_iff_as_set_subset(a, subsemigroup_closure(b))
        subsemigroup_subset(subsemigroup_closure(a), subsemigroup_closure(b))
        subsemigroup_closure_subset_iff_as_set_subset(b, subsemigroup_closure(a))
        subsemigroup_subset(subsemigroup_closure(b), subsemigroup_closure(a))
        subsemigroup_subset_antisymm(subsemigroup_closure(a), subsemigroup_closure(b))
        subsemigroup_closure(a) = subsemigroup_closure(b)
    }
}
