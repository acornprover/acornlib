// Concrete deepening: theorems about the concrete instances — the integers
// `Int` as a commutative ring and the rationals `Rat` as a field — obtained
// by instantiating the abstract algebra results of this package at the
// concrete types, together with the arithmetic of the rational number two
// and its inverse.

from int import Int, one_neq_zero, add_from_nat
from rat import Rat, add_pos_pos, one_is_pos, pos_ne_zero, pos_inverse
from algebra.ring.ring import Ring, mul_zero_left, mul_neg_one_left
from algebra.add_group import inverse_inverse
from algebra.field.field import Field, mul_inverse_right, mul_inverse_left, inverse_not_zero
from algebra.field.field_deep import fdiv, field_fdiv_mul_cancel, field_fdiv_inverse,
    field_mul_cancel_left, field_pow_inverse_nat
from algebra.ring.ring_axioms_deep import ring_sub_self, ring_pow_two
from algebra.ring.ring_theorems_deep import ring_two_mul
from nat import Nat, alt_suc_ne_zero, from_nat
numerals Int
numerals Nat

/// The negative of the integer one is not zero.
theorem int_neg_one_neq_zero {
    -Int.1 != Int.0
} by {
    if -Int.1 = Int.0 {
        inverse_inverse(Int.1)
        --Int.1 = Int.1
        -(-Int.1) = -Int.0
        -Int.0 = Int.0
        Int.1 = Int.0
        one_neq_zero
        false
    }
}

/// One plus one is not zero as an integer.
theorem int_two_ne_zero {
    Int.1 + Int.1 != Int.0
} by {
    add_from_nat(Nat.1, Nat.1)
    Int.from_nat(Nat.1) + Int.from_nat(Nat.1) = Int.from_nat(Nat.1 + Nat.1)
    alt_suc_ne_zero(Nat.1)
    Nat.2 != Nat.0
    Int.from_nat(Nat.2) != Int.from_nat(Nat.0)
    Int.1 + Int.1 = Int.from_nat(Nat.2)
    Int.0 = Int.from_nat(Nat.0)
    Int.1 + Int.1 != Int.0
}

/// Doubling an integer duplicates it.
theorem int_two_mul(a: Int) {
    (Int.1 + Int.1) * a = a + a
} by {
    ring_two_mul(a)
    (Int.1 + Int.1) * a = a + a
}

/// The double negation of the integer one is one.
theorem int_neg_neg_one {
    --Int.1 = Int.1
} by {
    inverse_inverse(Int.1)
    --Int.1 = Int.1
}

/// The natural embedding of two into the integers is one plus one.
theorem int_from_nat_two {
    from_nat[Int](2) = Int.1 + Int.1
} by {
    from_nat[Int](Nat.0.suc.suc) = from_nat[Int](Nat.0.suc) + Int.1
    from_nat[Int](Nat.0.suc) = from_nat[Int](Nat.0) + Int.1
    from_nat[Int](Nat.0) = Int.0
    from_nat[Int](2) = Int.1 + Int.1
}

/// The square of an integer is the product with itself.
theorem int_pow_two(a: Int) {
    a.pow(2) = a * a
} by {
    ring_pow_two(a)
    a.pow(2) = a * a
}

/// The square of a self-difference is zero.
theorem int_sub_self_pow_two(a: Int) {
    (a - a).pow(2) = Int.0
} by {
    ring_sub_self(a)
    a - a = Int.0
    (a - a).pow(2) = Int.0.pow(2)
    ring_pow_two(Int.0)
    Int.0.pow(2) = Int.0 * Int.0
    mul_zero_left(Int.0)
    Int.0 * Int.0 = Int.0
    (a - a).pow(2) = Int.0
}

/// Multiplying an integer by negative one negates it.
theorem int_mul_neg_one(a: Int) {
    a * -Int.1 = -a
} by {
    mul_neg_one_left(a)
    -Int.1 * a = -a
    a * -Int.1 = -Int.1 * a
    a * -Int.1 = -a
}

/// The inverse of the rational two multiplied by two is one.
theorem rat_two_inverse_mul_two {
    Rat.2.inverse * Rat.2 = Rat.1
} by {
    mul_inverse_left(Rat.2)
    Rat.2 != Rat.0
    Rat.2.inverse * Rat.2 = Rat.1
}

/// Two multiplied by its inverse is one.
theorem rat_two_mul_inverse {
    Rat.2 * Rat.2.inverse = Rat.1
} by {
    mul_inverse_right(Rat.2)
    Rat.2 != Rat.0
    Rat.2 * Rat.2.inverse = Rat.1
}

/// The inverse of the inverse of two is two.
theorem rat_two_inverse_inverse {
    Rat.2.inverse.inverse = Rat.2
} by {
    inverse_inverse(Rat.2)
    Rat.2.inverse.inverse = Rat.2
}

/// The inverse of two is not zero.
theorem rat_two_inverse_ne_zero {
    Rat.2.inverse != Rat.0
} by {
    inverse_not_zero(Rat.2)
    Rat.2 != Rat.0
    Rat.2.inverse != Rat.0
}

/// Dividing one by two gives the inverse of two.
theorem rat_fdiv_one_two {
    fdiv(Rat.1, Rat.2) = Rat.2.inverse
} by {
    field_fdiv_inverse(Rat.2)
    fdiv(Rat.1, Rat.2) = Rat.2.inverse
}

/// Multiplying the quotient one-over-two by two gives one.
theorem rat_fdiv_one_two_cancel {
    fdiv(Rat.1, Rat.2) * Rat.2 = Rat.1
} by {
    field_fdiv_mul_cancel(Rat.1, Rat.2)
    Rat.2 != Rat.0
    fdiv(Rat.1, Rat.2) * Rat.2 = Rat.1
}

/// One plus one is positive as a rational.
theorem rat_one_plus_one_pos {
    (Rat.1 + Rat.1).is_positive
} by {
    one_is_pos
    Rat.1.is_positive
    add_pos_pos(Rat.1, Rat.1)
    Rat.1.is_positive and Rat.1.is_positive implies (Rat.1 + Rat.1).is_positive
    (Rat.1 + Rat.1).is_positive
}

/// One plus one is not zero as a rational.
theorem rat_one_plus_one_ne_zero {
    Rat.1 + Rat.1 != Rat.0
} by {
    pos_ne_zero(Rat.1 + Rat.1)
    (Rat.1 + Rat.1).is_positive implies Rat.1 + Rat.1 != Rat.0
    rat_one_plus_one_pos
    Rat.1 + Rat.1 != Rat.0
}

/// The inverse of one plus one is positive as a rational.
theorem rat_one_plus_one_inverse_pos {
    (Rat.1 + Rat.1).inverse.is_positive
} by {
    pos_inverse(Rat.1 + Rat.1)
    (Rat.1 + Rat.1).is_positive implies (Rat.1 + Rat.1).inverse.is_positive
    rat_one_plus_one_pos
    (Rat.1 + Rat.1).inverse.is_positive
}

/// Multiplying by a nonzero rational is left-cancellative.
theorem rat_mul_cancel_left(a: Rat, b: Rat, c: Rat) {
    a != Rat.0 and a * b = a * c implies b = c
} by {
    if a != Rat.0 and a * b = a * c {
        field_mul_cancel_left(a, b, c)
        b = c
    }
}

/// The square of the inverse of two is the inverse of the square of two.
theorem rat_two_inverse_pow_two {
    Rat.2.inverse.pow(2) = Rat.2.pow(2).inverse
} by {
    field_pow_inverse_nat(Rat.2, Nat.2)
    Rat.2.inverse.pow(2) = Rat.2.pow(2).inverse
}
