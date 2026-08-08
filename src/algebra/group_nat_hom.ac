from algebra.add_monoid import AddMonoid, AddMonoidHom, is_add_monoid_hom,
    add_monoid_hom_zero, add_monoid_hom_add, add_monoid_hom_ext
from algebra.monoid.monoid import Monoid, MonoidHom, is_monoid_hom, monoid_hom_one,
    monoid_hom_mul, monoid_hom_ext
from nat import Nat, add_zero_right, add_one_right, add_suc_right, pow_one, pow_add, pow_zero
from data.basic.functions import function_extensionality

/// The canonical repeated-addition map from Nat into an additive monoid.
define nat_multiples_map[M: AddMonoid](x: M, n: Nat) -> M {
    match n {
        Nat.zero {
            M.0
        }
        Nat.suc(k) {
            nat_multiples_map(x, k) + x
        }
    }
}

/// Repeated addition by zero is zero.
theorem nat_multiples_zero[M: AddMonoid](x: M) {
    nat_multiples_map(x, Nat.0) = M.0
}

/// Repeated addition by one is the underlying element.
theorem nat_multiples_one[M: AddMonoid](x: M) {
    nat_multiples_map(x, Nat.1) = x
} by {
    nat_multiples_map(x, Nat.1) = nat_multiples_map(x, Nat.0) + x
    nat_multiples_map(x, Nat.0) = M.0
    M.0 + x = x
}

/// Repeated addition is additive in the natural-number argument.
theorem nat_multiples_add[M: AddMonoid](x: M, n: Nat, m: Nat) {
    nat_multiples_map(x, n + m) = nat_multiples_map(x, n) + nat_multiples_map(x, m)
} by {
    define p(k: Nat) -> Bool {
        nat_multiples_map(x, n + k) = nat_multiples_map(x, n) + nat_multiples_map(x, k)
    }

    add_zero_right(n)
    n + Nat.0 = n
    nat_multiples_map(x, Nat.0) = M.0
    nat_multiples_map(x, n) + M.0 = nat_multiples_map(x, n)
    p(Nat.0)

    forall(k: Nat) {
        if p(k) {
            add_suc_right(n, k)
            n + k.suc = (n + k).suc
            nat_multiples_map(x, n + k.suc) = nat_multiples_map(x, n + k) + x
            nat_multiples_map(x, n + k) = nat_multiples_map(x, n) + nat_multiples_map(x, k)
            nat_multiples_map(x, n + k.suc) =
                (nat_multiples_map(x, n) + nat_multiples_map(x, k)) + x
            nat_multiples_map(x, k.suc) = nat_multiples_map(x, k) + x
            nat_multiples_map(x, n) + nat_multiples_map(x, k.suc) =
                nat_multiples_map(x, n) + (nat_multiples_map(x, k) + x)
            nat_multiples_map(x, n) + (nat_multiples_map(x, k) + x) =
                (nat_multiples_map(x, n) + nat_multiples_map(x, k)) + x
            p(k.suc)
        }
    }
    p(m)
}

/// The repeated-addition map preserves zero and addition.
theorem nat_multiples_map_is_add_monoid_hom[M: AddMonoid](x: M) {
    is_add_monoid_hom(nat_multiples_map[M](x))
} by {
    nat_multiples_zero(x)
    nat_multiples_map[M](x, Nat.0) = M.0
    forall(a: Nat, b: Nat) {
        nat_multiples_add(x, a, b)
    }
}

/// Additive homomorphisms from Nat are determined by the image of one.
theorem ext_nat_prime[M: AddMonoid](f: AddMonoidHom[Nat, M], g: AddMonoidHom[Nat, M]) {
    f.hom(Nat.1) = g.hom(Nat.1) implies f = g
} by {
    if f.hom(Nat.1) = g.hom(Nat.1) {
        define p(n: Nat) -> Bool {
            f.hom(n) = g.hom(n)
        }

        add_monoid_hom_zero(f)
        add_monoid_hom_zero(g)
        f.hom(Nat.0) = M.0
        g.hom(Nat.0) = M.0
        p(Nat.0)

        forall(k: Nat) {
            if p(k) {
                add_one_right(k)
                k + Nat.1 = k.suc
                add_monoid_hom_add(f, k, Nat.1)
                add_monoid_hom_add(g, k, Nat.1)
                f.hom(k + Nat.1) = f.hom(k) + f.hom(Nat.1)
                g.hom(k + Nat.1) = g.hom(k) + g.hom(Nat.1)
                f.hom(k.suc) = f.hom(k) + f.hom(Nat.1)
                g.hom(k.suc) = g.hom(k) + g.hom(Nat.1)
                f.hom(k) = g.hom(k)
                f.hom(Nat.1) = g.hom(Nat.1)
                f.hom(k.suc) = g.hom(k.suc)
                p(k.suc)
            }
        }

        forall(n: Nat) {
            p(n)
            f.hom(n) = g.hom(n)
        }
        add_monoid_hom_ext(f, g)
        f = g
    }
}

/// Method-style wrapper for `ext_nat_prime`.
theorem add_monoid_hom_ext_nat[M: AddMonoid](f: AddMonoidHom[Nat, M], g: AddMonoidHom[Nat, M]) {
    f.hom(Nat.1) = g.hom(Nat.1) implies f = g
} by {
    ext_nat_prime(f, g)
}

/// Additive homomorphism from Nat generated by one element.
let multiplesHom[M: AddMonoid](x: M) -> result: AddMonoidHom[Nat, M] satisfy {
    AddMonoidHom.new(nat_multiples_map[M](x)) = Option.some(result)
} by {
    nat_multiples_map_is_add_monoid_hom(x)
}

/// The generated additive homomorphism evaluates to repeated addition.
theorem multiplesHom_apply[M: AddMonoid](x: M, n: Nat) {
    multiplesHom(x).hom(n) = nat_multiples_map(x, n)
}

/// Homomorphisms from Nat are generated by their value on one.
theorem add_monoid_hom_apply_nat[M: AddMonoid](f: AddMonoidHom[Nat, M], n: Nat) {
    f.hom(n) = nat_multiples_map(f.hom(Nat.1), n)
} by {
    let generated = multiplesHom[M](f.hom(Nat.1))
    multiplesHom_apply(f.hom(Nat.1), Nat.1)
    nat_multiples_one(f.hom(Nat.1))
    generated.hom(Nat.1) = f.hom(Nat.1)
    ext_nat_prime(generated, f)
    generated = f
    multiplesHom_apply(f.hom(Nat.1), n)
}
