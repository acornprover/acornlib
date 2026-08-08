/// Multiplicative homomorphisms preserve and reflect semiconjugacy and commutation.

from data.basic.functions import is_injective_fn, injective_fn_eq
from algebra.monoid.monoid import Monoid, MonoidHom, monoid_hom_mul
from algebra.group.group_semiconj import semiconj_by, semiconj_by_mul_eq
from algebra.group.group_commute import commute, commute_eq, commute_intro

/// A monoid homomorphism preserves semiconjugacy.
theorem semiconj_by_map[M: Monoid, N: Monoid](f: MonoidHom[M, N], a: M, x: M, y: M) {
    semiconj_by(a, x, y) implies semiconj_by(f.hom(a), f.hom(x), f.hom(y))
} by {
    if semiconj_by(a, x, y) {
        semiconj_by_mul_eq(a, x, y)
        a * x = y * a
        monoid_hom_mul(f, a, x)
        monoid_hom_mul(f, y, a)
        f.hom(a * x) = f.hom(a) * f.hom(x)
        f.hom(y * a) = f.hom(y) * f.hom(a)
        f.hom(a * x) = f.hom(y * a)
        f.hom(a) * f.hom(x) = f.hom(y) * f.hom(a)
        semiconj_by(f.hom(a), f.hom(x), f.hom(y))
    }
}

/// A monoid homomorphism preserves commutation.
theorem commute_map[M: Monoid, N: Monoid](f: MonoidHom[M, N], x: M, y: M) {
    commute(x, y) implies commute(f.hom(x), f.hom(y))
} by {
    if commute(x, y) {
        commute_eq(x, y)
        x * y = y * x
        semiconj_by(x, y, y)
        semiconj_by_map(f, x, y, y)
        semiconj_by(f.hom(x), f.hom(y), f.hom(y))
        semiconj_by_mul_eq(f.hom(x), f.hom(y), f.hom(y))
        f.hom(x) * f.hom(y) = f.hom(y) * f.hom(x)
        commute_intro[N](f.hom(x), f.hom(y))
        commute[N](f.hom(x), f.hom(y))
    }
}

/// An injective monoid homomorphism reflects semiconjugacy.
theorem semiconj_by_of_map[M: Monoid, N: Monoid](f: MonoidHom[M, N], a: M, x: M, y: M) {
    is_injective_fn(f.hom) and semiconj_by(f.hom(a), f.hom(x), f.hom(y))
    implies semiconj_by(a, x, y)
} by {
    if is_injective_fn(f.hom) and semiconj_by(f.hom(a), f.hom(x), f.hom(y)) {
        semiconj_by_mul_eq(f.hom(a), f.hom(x), f.hom(y))
        f.hom(a) * f.hom(x) = f.hom(y) * f.hom(a)
        monoid_hom_mul(f, a, x)
        monoid_hom_mul(f, y, a)
        f.hom(a * x) = f.hom(a) * f.hom(x)
        f.hom(y * a) = f.hom(y) * f.hom(a)
        f.hom(a * x) = f.hom(y * a)
        injective_fn_eq(f.hom, a * x, y * a)
        a * x = y * a
        semiconj_by(a, x, y)
    }
}

/// An injective monoid homomorphism reflects commutation.
theorem commute_of_map[M: Monoid, N: Monoid](f: MonoidHom[M, N], x: M, y: M) {
    is_injective_fn(f.hom) and commute(f.hom(x), f.hom(y)) implies commute(x, y)
} by {
    if is_injective_fn(f.hom) and commute(f.hom(x), f.hom(y)) {
        commute_eq(f.hom(x), f.hom(y))
        f.hom(x) * f.hom(y) = f.hom(y) * f.hom(x)
        semiconj_by(f.hom(x), f.hom(y), f.hom(y))
        semiconj_by_of_map(f, x, y, y)
        semiconj_by(x, y, y)
        semiconj_by_mul_eq(x, y, y)
        x * y = y * x
        commute_intro[M](x, y)
        commute[M](x, y)
    }
}

/// For an injective monoid homomorphism, mapped semiconjugacy is equivalent to semiconjugacy.
theorem semiconj_by_map_iff[M: Monoid, N: Monoid](f: MonoidHom[M, N], a: M, x: M, y: M) {
    is_injective_fn(f.hom) implies
    semiconj_by(f.hom(a), f.hom(x), f.hom(y)) = semiconj_by(a, x, y)
} by {
    if is_injective_fn(f.hom) {
        if semiconj_by(f.hom(a), f.hom(x), f.hom(y)) {
            semiconj_by_of_map(f, a, x, y)
            semiconj_by(a, x, y)
            semiconj_by(f.hom(a), f.hom(x), f.hom(y)) = semiconj_by(a, x, y)
        } else {
            if semiconj_by(a, x, y) {
                semiconj_by_map(f, a, x, y)
                false
            }
            semiconj_by(f.hom(a), f.hom(x), f.hom(y)) = semiconj_by(a, x, y)
        }
    }
}

/// For an injective monoid homomorphism, mapped commutation is equivalent to commutation.
theorem commute_map_iff[M: Monoid, N: Monoid](f: MonoidHom[M, N], x: M, y: M) {
    is_injective_fn(f.hom) implies commute(f.hom(x), f.hom(y)) = commute(x, y)
} by {
    if is_injective_fn(f.hom) {
        if commute(f.hom(x), f.hom(y)) {
            commute_of_map(f, x, y)
            commute(x, y)
            commute(f.hom(x), f.hom(y)) = commute(x, y)
        } else {
            if commute(x, y) {
                commute_map(f, x, y)
                false
            }
            commute(f.hom(x), f.hom(y)) = commute(x, y)
        }
    }
}
