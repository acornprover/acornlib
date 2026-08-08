from semiring import Semiring
from lte import LTE
from data.basic.relation_basic import is_reflexive, is_transitive, is_antisymmetric
from order import PartialOrder
from lattice import Meet, MeetSemilattice
from algebra.subsemiring import Subsemiring, subsemiring_subset, subsemiring_subset_refl,
    subsemiring_subset_trans, subsemiring_subset_antisymm,
    subsemiring_intersection_subset_left_relation,
    subsemiring_intersection_subset_right_relation,
    subsemiring_subset_intersection_of_subset_left_right

/// Subsemiring inclusion gives the generic `LTE` relation.
instance Subsemiring[S: Semiring]: LTE {
    define lte(self, other: Subsemiring[S]) -> Bool {
        subsemiring_subset(self, other)
    }
}

/// The generic order on subsemirings is subsemiring inclusion.
theorem subsemiring_lte_eq_subset[S: Semiring](a: Subsemiring[S], b: Subsemiring[S]) {
    (a <= b) = subsemiring_subset(a, b)
}

/// The packaged subsemiring order is reflexive.
theorem subsemiring_instance_lte_refl[S: Semiring] {
    is_reflexive(LTE.lte[Subsemiring[S]])
} by {
    forall(a: Subsemiring[S]) {
        subsemiring_subset_refl(a)
        subsemiring_lte_eq_subset(a, a)
        a <= a
    }
}

/// The packaged subsemiring order is transitive.
theorem subsemiring_instance_lte_trans[S: Semiring] {
    is_transitive(LTE.lte[Subsemiring[S]])
} by {
    is_transitive(LTE.lte[Subsemiring[S]]) = forall(a: Subsemiring[S], b: Subsemiring[S], c: Subsemiring[S]) {
        a <= b and b <= c implies a <= c
    }
    forall(a: Subsemiring[S], b: Subsemiring[S], c: Subsemiring[S]) {
        if a <= b and b <= c {
            subsemiring_lte_eq_subset(a, b)
            subsemiring_lte_eq_subset(b, c)
            subsemiring_subset(a, b)
            subsemiring_subset(b, c)
            subsemiring_subset_trans(a, b, c)
            subsemiring_subset(a, c)
            subsemiring_lte_eq_subset(a, c)
            a <= c
        }
    }
}

/// The packaged subsemiring order is antisymmetric.
theorem subsemiring_instance_lte_antisymm[S: Semiring] {
    is_antisymmetric(LTE.lte[Subsemiring[S]])
} by {
    forall(a: Subsemiring[S], b: Subsemiring[S]) {
        if a <= b and b <= a {
            subsemiring_lte_eq_subset(a, b)
            subsemiring_lte_eq_subset(b, a)
            subsemiring_subset(a, b)
            subsemiring_subset(b, a)
            subsemiring_subset_antisymm(a, b)
            a = b
        }
    }
}

/// Subsemirings are partially ordered by inclusion.
instance Subsemiring[S: Semiring]: PartialOrder

/// Subsemiring intersection gives the generic meet operation.
instance Subsemiring[S: Semiring]: Meet {
    define meet(self, other: Subsemiring[S]) -> Subsemiring[S] {
        self.intersection(other)
    }
}

/// The generic meet on subsemirings is intersection.
theorem subsemiring_meet_eq_intersection[S: Semiring](a: Subsemiring[S], b: Subsemiring[S]) {
    a.meet(b) = a.intersection(b)
}

/// A subsemiring meet is below its left argument.
theorem subsemiring_instance_meet_lte_left[S: Semiring](a: Subsemiring[S], b: Subsemiring[S]) {
    a.meet(b) <= a
} by {
    subsemiring_meet_eq_intersection(a, b)
    subsemiring_intersection_subset_left_relation(a, b)
    subsemiring_subset(a.intersection(b), a)
    subsemiring_lte_eq_subset(a.intersection(b), a)
    a.intersection(b) <= a
    a.meet(b) <= a
}

/// A subsemiring meet is below its right argument.
theorem subsemiring_instance_meet_lte_right[S: Semiring](a: Subsemiring[S], b: Subsemiring[S]) {
    a.meet(b) <= b
} by {
    subsemiring_meet_eq_intersection(a, b)
    subsemiring_intersection_subset_right_relation(a, b)
    subsemiring_subset(a.intersection(b), b)
    subsemiring_lte_eq_subset(a.intersection(b), b)
    a.intersection(b) <= b
    a.meet(b) <= b
}

/// Any common lower bound is below the subsemiring meet.
theorem subsemiring_instance_lte_meet_of_bounds[S: Semiring](
    c: Subsemiring[S],
    a: Subsemiring[S],
    b: Subsemiring[S]
) {
    c <= a and c <= b implies c <= a.meet(b)
} by {
    if c <= a and c <= b {
        subsemiring_lte_eq_subset(c, a)
        subsemiring_lte_eq_subset(c, b)
        subsemiring_subset(c, a)
        subsemiring_subset(c, b)
        subsemiring_subset_intersection_of_subset_left_right(c, a, b)
        subsemiring_subset(c, a.intersection(b))
        subsemiring_lte_eq_subset(c, a.intersection(b))
        c <= a.intersection(b)
        subsemiring_meet_eq_intersection(a, b)
        c <= a.meet(b)
    }
}

/// Subsemirings form a meet-semilattice under intersection.
instance Subsemiring[S: Semiring]: MeetSemilattice
