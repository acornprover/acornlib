from nat import Nat
from algebra.add_group import AddGroup, left_cancel, right_cancel
from order import PartialOrder, LinearOrder
from order_relation import lt_relation_is_transitive
from order import is_monotone, is_antitone, is_strict_monotone, is_strict_antitone,
    is_order_embedding, order_embedding_is_strict_monotone, order_embedding_apply_le,
    order_embedding_apply_lt, order_embedding_apply_ge, order_embedding_apply_gt,
    order_embedding_reflects_le, order_embedding_reflects_lt, order_embedding_reflects_ge,
    order_embedding_reflects_gt
from order_iso import OrderIso, OrderDualIso, is_order_iso_pair, is_order_dual_iso_pair,
    order_iso_new_map, order_iso_new_inv, order_iso_le_iff_le, order_iso_lt_iff_lt,
    order_iso_ge_iff_ge, order_iso_gt_iff_gt, order_dual_iso_new_map, order_dual_iso_new_inv
from data.basic.relation_basic import antisymmetric_eq, transitive_step

/// A left-ordered group is a group with a left-invariant order.
/// This means that if `a <= b`, then `c + a <= c + b` for any c in the group.
typeclass G: AddLeftOrderedGroup extends AddGroup, LinearOrder {
    /// Left addition preserves the order relation: if `a ≤ b`, then `c + a ≤ c + b`.
    left_invariance(a: G, b: G, c: G) {
        a <= b implies c + a <= c + b
    }
}

theorem lt_trans[G: AddLeftOrderedGroup](a: G, b: G, c: G) {
    a < b and b < c implies a < c
} by {
    if a < b and b < c {
        lt_relation_is_transitive[G]
        transitive_step(G.lt, a, b, c)
        a < c
    }
}

/// An ordered group has both left and right invariance.
typeclass G: AddOrderedGroup extends AddLeftOrderedGroup {
    /// Right addition preserves the order relation: if `a ≤ b`, then `a + c ≤ b + c`.
    right_invariance(a: G, b: G, c: G) {
        a <= b implies a + c <= b + c
    }
}

theorem negative_of_nonnegative[G: AddLeftOrderedGroup](a: G) {
    G.0 <= a implies -a <= G.0
}

theorem negative_of_negative[G: AddLeftOrderedGroup](a: G) {
    a <= G.0 implies G.0 <= -a
}

theorem add_inequality[F: AddOrderedGroup](a: F, b: F, c: F, d: F) {
    a <= b and c <= d implies a + c <= b + d
} by {
    if a <= b and c <= d {
        b + c <= b + d
        transitive_step(F.lte, a + c, b + c, b + d)
        a + c <= b + d
    }
}

theorem minus_flips_inequality[F: AddOrderedGroup](a: F, b: F) {
    a <= b implies -b <= -a
} by {
    F.0 <= b + -a
    F.0 <= b + -a implies -b + F.0 <= -b + (b + -a)
    -b + (b + -a) = -b + b + -a
}

/// The map obtained by adding a fixed element on the left.
define add_left_map[G: AddLeftOrderedGroup](c: G, x: G) -> G {
    c + x
}

/// The map obtained by adding a fixed element on the right.
define add_right_map[G: AddOrderedGroup](c: G, x: G) -> G {
    x + c
}

/// Left translation is the corresponding addition.
theorem add_left_map_apply[G: AddLeftOrderedGroup](c: G, x: G) {
    add_left_map(c, x) = c + x
}

/// Right translation is the corresponding addition.
theorem add_right_map_apply[G: AddOrderedGroup](c: G, x: G) {
    add_right_map(c, x) = x + c
}

/// Left translation preserves the non-strict order.
theorem add_left_map_is_monotone[G: AddLeftOrderedGroup](c: G) {
    is_monotone(add_left_map(c))
} by {
    forall(x: G, y: G) {
        if x <= y {
            c + x <= c + y
            add_left_map(c, x) <= add_left_map(c, y)
        }
    }
}

/// Right translation preserves the non-strict order.
theorem add_right_map_is_monotone[G: AddOrderedGroup](c: G) {
    is_monotone(add_right_map(c))
} by {
    forall(x: G, y: G) {
        if x <= y {
            x + c <= y + c
            add_right_map(c, x) <= add_right_map(c, y)
        }
    }
}

/// Left translation reflects the non-strict order.
theorem add_left_map_reflects_le[G: AddLeftOrderedGroup](c: G, x: G, y: G) {
    add_left_map(c, x) <= add_left_map(c, y) implies x <= y
} by {
    if add_left_map(c, x) <= add_left_map(c, y) {
        add_left_map(c, x) = c + x
        add_left_map(c, y) = c + y
        -c + (c + x) <= -c + (c + y)
        -c + (c + x) = -c + c + x
        -c + (c + y) = -c + c + y
        -c + c = G.0
        x <= y
    }
}

/// Right translation reflects the non-strict order.
theorem add_right_map_reflects_le[G: AddOrderedGroup](c: G, x: G, y: G) {
    add_right_map(c, x) <= add_right_map(c, y) implies x <= y
} by {
    if add_right_map(c, x) <= add_right_map(c, y) {
        add_right_map(c, x) = x + c
        add_right_map(c, y) = y + c
        (x + c) + -c <= (y + c) + -c
        x + c + -c = x + (c + -c)
        y + c + -c = y + (c + -c)
        c + -c = G.0
        x <= y
    }
}

/// Left translation preserves and reflects the non-strict order.
theorem add_left_map_is_order_embedding[G: AddLeftOrderedGroup](c: G) {
    is_order_embedding(add_left_map(c))
} by {
    forall(x: G, y: G) {
        if add_left_map(c, x) <= add_left_map(c, y) {
            add_left_map_reflects_le(c, x, y)
            x <= y
        }
        if x <= y {
            add_left_map_is_monotone(c)
            is_monotone(add_left_map(c))
            add_left_map(c, x) <= add_left_map(c, y)
        }
        add_left_map(c, x) <= add_left_map(c, y) = (x <= y)
    }
}

/// Right translation preserves and reflects the non-strict order.
theorem add_right_map_is_order_embedding[G: AddOrderedGroup](c: G) {
    is_order_embedding(add_right_map(c))
} by {
    forall(x: G, y: G) {
        if add_right_map(c, x) <= add_right_map(c, y) {
            add_right_map_reflects_le(c, x, y)
            x <= y
        }
        if x <= y {
            add_right_map_is_monotone(c)
            is_monotone(add_right_map(c))
            add_right_map(c, x) <= add_right_map(c, y)
        }
        add_right_map(c, x) <= add_right_map(c, y) = (x <= y)
    }
}

/// Left translation preserves strict order.
theorem add_left_map_is_strict_monotone[G: AddLeftOrderedGroup](c: G) {
    is_strict_monotone(add_left_map(c))
} by {
    add_left_map_is_order_embedding(c)
    order_embedding_is_strict_monotone(add_left_map(c))
}

/// Right translation preserves strict order.
theorem add_right_map_is_strict_monotone[G: AddOrderedGroup](c: G) {
    is_strict_monotone(add_right_map(c))
} by {
    add_right_map_is_order_embedding(c)
    order_embedding_is_strict_monotone(add_right_map(c))
}

/// Left translation by the additive inverse is a left inverse of left translation.
theorem add_left_map_left_inverse[G: AddLeftOrderedGroup](c: G, x: G) {
    add_left_map(-c, add_left_map(c, x)) = x
} by {
    -c + (c + x) = -c + c + x
}

/// Left translation is a left inverse of translation by the additive inverse.
theorem add_left_map_right_inverse[G: AddLeftOrderedGroup](c: G, x: G) {
    add_left_map(c, add_left_map(-c, x)) = x
} by {
}

/// Right translation by the additive inverse is a left inverse of right translation.
theorem add_right_map_left_inverse[G: AddOrderedGroup](c: G, x: G) {
    add_right_map(-c, add_right_map(c, x)) = x
} by {
    (x + c) + -c = x + (c + -c)
}

/// Right translation is a left inverse of translation by the additive inverse.
theorem add_right_map_right_inverse[G: AddOrderedGroup](c: G, x: G) {
    add_right_map(c, add_right_map(-c, x)) = x
} by {
}

/// Left translation and translation by the additive inverse form an order isomorphism pair.
theorem add_left_map_is_order_iso_pair[G: AddLeftOrderedGroup](c: G) {
    is_order_iso_pair(add_left_map(c), add_left_map(-c))
} by {
    let f = add_left_map(c)
    let g = add_left_map(-c)
    let left_inverse: Bool = forall(x: G) {
        g(f(x)) = x
    }
    let right_inverse: Bool = forall(y: G) {
        f(g(y)) = y
    }
    forall(x: G) {
        add_left_map_left_inverse(c, x)
        g(f(x)) = x
    }
    forall(y: G) {
        add_left_map_right_inverse(c, y)
        f(g(y)) = y
    }
    right_inverse
    add_left_map_is_order_embedding(c)
    add_left_map_is_order_embedding(-c)
    left_inverse and right_inverse
    left_inverse and right_inverse and is_order_embedding(f)
    is_order_iso_pair(f, g)
}

/// Right translation and translation by the additive inverse form an order isomorphism pair.
theorem add_right_map_is_order_iso_pair[G: AddOrderedGroup](c: G) {
    is_order_iso_pair(add_right_map(c), add_right_map(-c))
} by {
    let f = add_right_map(c)
    let g = add_right_map(-c)
    let left_inverse: Bool = forall(x: G) {
        g(f(x)) = x
    }
    let right_inverse: Bool = forall(y: G) {
        f(g(y)) = y
    }
    forall(x: G) {
        add_right_map_left_inverse(c, x)
        g(f(x)) = x
    }
    forall(y: G) {
        add_right_map_right_inverse(c, y)
        f(g(y)) = y
    }
    right_inverse
    add_right_map_is_order_embedding(c)
    add_right_map_is_order_embedding(-c)
    left_inverse and right_inverse
    left_inverse and right_inverse and is_order_embedding(f)
    is_order_iso_pair(f, g)
}

/// Left translation by a fixed element as an order isomorphism.
let add_left_order_iso[G: AddLeftOrderedGroup](c: G) -> result: OrderIso[G, G] satisfy {
    result.map = add_left_map(c) and result.inv = add_left_map(-c)
} by {
    add_left_map_is_order_iso_pair(c)
    let e: OrderIso[G, G] satisfy {
        OrderIso[G, G].new(add_left_map(c), add_left_map(-c)) = Option.some(e)
    }
    order_iso_new_map(add_left_map(c), add_left_map(-c), e)
    order_iso_new_inv(add_left_map(c), add_left_map(-c), e)
}

/// The map of the left-translation order isomorphism is left translation.
theorem add_left_order_iso_map[G: AddLeftOrderedGroup](c: G) {
    add_left_order_iso(c).map = add_left_map(c)
} by {
    let e = add_left_order_iso(c)
    order_iso_new_map(add_left_map(c), add_left_map(-c), e)
}

/// The inverse of the left-translation order isomorphism is left translation by the additive inverse.
theorem add_left_order_iso_inv[G: AddLeftOrderedGroup](c: G) {
    add_left_order_iso(c).inv = add_left_map(-c)
} by {
    let e = add_left_order_iso(c)
    order_iso_new_inv(add_left_map(c), add_left_map(-c), e)
}

/// Right translation by a fixed element as an order isomorphism.
let add_right_order_iso[G: AddOrderedGroup](c: G) -> result: OrderIso[G, G] satisfy {
    result.map = add_right_map(c) and result.inv = add_right_map(-c)
} by {
    add_right_map_is_order_iso_pair(c)
    let e: OrderIso[G, G] satisfy {
        OrderIso[G, G].new(add_right_map(c), add_right_map(-c)) = Option.some(e)
    }
    order_iso_new_map(add_right_map(c), add_right_map(-c), e)
    order_iso_new_inv(add_right_map(c), add_right_map(-c), e)
}

/// The map of the right-translation order isomorphism is right translation.
theorem add_right_order_iso_map[G: AddOrderedGroup](c: G) {
    add_right_order_iso(c).map = add_right_map(c)
} by {
    let e = add_right_order_iso(c)
    order_iso_new_map(add_right_map(c), add_right_map(-c), e)
}

/// The inverse of the right-translation order isomorphism is right translation by the additive inverse.
theorem add_right_order_iso_inv[G: AddOrderedGroup](c: G) {
    add_right_order_iso(c).inv = add_right_map(-c)
} by {
    let e = add_right_order_iso(c)
    order_iso_new_inv(add_right_map(c), add_right_map(-c), e)
}

/// Adding a fixed element on the left preserves a non-strict comparison.
theorem add_le_add_left[G: AddLeftOrderedGroup](a: G, b: G, c: G) {
    a <= b implies c + a <= c + b
} by {
    if a <= b {
        add_left_map_is_order_embedding(c)
        order_embedding_apply_le(add_left_map(c), a, b)
        c + a <= c + b
    }
}

/// Adding a fixed element on the right preserves a non-strict comparison.
theorem add_le_add_right[G: AddOrderedGroup](a: G, b: G, c: G) {
    a <= b implies a + c <= b + c
} by {
    if a <= b {
        add_right_map_is_order_embedding(c)
        order_embedding_apply_le(add_right_map(c), a, b)
        a + c <= b + c
    }
}

/// Adding a fixed element on the left preserves a strict comparison.
theorem add_lt_add_left[G: AddLeftOrderedGroup](a: G, b: G, c: G) {
    a < b implies c + a < c + b
} by {
    if a < b {
        add_left_map_is_order_embedding(c)
        order_embedding_apply_lt(add_left_map(c), a, b)
        add_left_map(c, a) < add_left_map(c, b)
        c + a < c + b
    }
}

/// Adding a fixed element on the right preserves a strict comparison.
theorem add_lt_add_right[G: AddOrderedGroup](a: G, b: G, c: G) {
    a < b implies a + c < b + c
} by {
    if a < b {
        add_right_map_is_order_embedding(c)
        order_embedding_apply_lt(add_right_map(c), a, b)
        add_right_map(c, a) < add_right_map(c, b)
        a + c < b + c
    }
}

/// Adding a fixed element on the left preserves a reverse non-strict comparison.
theorem add_ge_add_left[G: AddLeftOrderedGroup](a: G, b: G, c: G) {
    a >= b implies c + a >= c + b
} by {
    if a >= b {
        add_left_map_is_order_embedding(c)
        order_embedding_apply_ge(add_left_map(c), a, b)
        add_left_map(c, a) >= add_left_map(c, b)
        c + a >= c + b
    }
}

/// Adding a fixed element on the right preserves a reverse non-strict comparison.
theorem add_ge_add_right[G: AddOrderedGroup](a: G, b: G, c: G) {
    a >= b implies a + c >= b + c
} by {
    if a >= b {
        add_right_map_is_order_embedding(c)
        order_embedding_apply_ge(add_right_map(c), a, b)
        add_right_map(c, a) >= add_right_map(c, b)
        a + c >= b + c
    }
}

/// Adding a fixed element on the left preserves a strict reverse comparison.
theorem add_gt_add_left[G: AddLeftOrderedGroup](a: G, b: G, c: G) {
    a > b implies c + a > c + b
} by {
    if a > b {
        add_left_map_is_order_embedding(c)
        order_embedding_apply_gt(add_left_map(c), a, b)
        add_left_map(c, a) > add_left_map(c, b)
        c + a > c + b
    }
}

/// Adding a fixed element on the right preserves a strict reverse comparison.
theorem add_gt_add_right[G: AddOrderedGroup](a: G, b: G, c: G) {
    a > b implies a + c > b + c
} by {
    if a > b {
        add_right_map_is_order_embedding(c)
        order_embedding_apply_gt(add_right_map(c), a, b)
        add_right_map(c, a) > add_right_map(c, b)
        a + c > b + c
    }
}

/// A non-strict comparison after adding on the left reflects to the original comparison.
theorem le_of_add_le_add_left[G: AddLeftOrderedGroup](a: G, b: G, c: G) {
    c + a <= c + b implies a <= b
} by {
    if c + a <= c + b {
        add_left_map_reflects_le(c, a, b)
        a <= b
    }
}

/// A non-strict comparison after adding on the right reflects to the original comparison.
theorem le_of_add_le_add_right[G: AddOrderedGroup](a: G, b: G, c: G) {
    a + c <= b + c implies a <= b
} by {
    if a + c <= b + c {
        add_right_map_reflects_le(c, a, b)
        a <= b
    }
}

/// A strict comparison after adding on the left reflects to the original comparison.
theorem lt_of_add_lt_add_left[G: AddLeftOrderedGroup](a: G, b: G, c: G) {
    c + a < c + b implies a < b
} by {
    if c + a < c + b {
        add_left_map_is_order_embedding(c)
        order_embedding_reflects_lt(add_left_map(c), a, b)
        a < b
    }
}

/// A strict comparison after adding on the right reflects to the original comparison.
theorem lt_of_add_lt_add_right[G: AddOrderedGroup](a: G, b: G, c: G) {
    a + c < b + c implies a < b
} by {
    if a + c < b + c {
        add_right_map_is_order_embedding(c)
        order_embedding_reflects_lt(add_right_map(c), a, b)
        a < b
    }
}

/// A reverse non-strict comparison after adding on the left reflects to the original comparison.
theorem ge_of_add_ge_add_left[G: AddLeftOrderedGroup](a: G, b: G, c: G) {
    c + a >= c + b implies a >= b
} by {
    if c + a >= c + b {
        add_left_map_is_order_embedding(c)
        order_embedding_reflects_ge(add_left_map(c), a, b)
        a >= b
    }
}

/// A reverse non-strict comparison after adding on the right reflects to the original comparison.
theorem ge_of_add_ge_add_right[G: AddOrderedGroup](a: G, b: G, c: G) {
    a + c >= b + c implies a >= b
} by {
    if a + c >= b + c {
        add_right_map_is_order_embedding(c)
        order_embedding_reflects_ge(add_right_map(c), a, b)
        a >= b
    }
}

/// A strict reverse comparison after adding on the left reflects to the original comparison.
theorem gt_of_add_gt_add_left[G: AddLeftOrderedGroup](a: G, b: G, c: G) {
    c + a > c + b implies a > b
} by {
    if c + a > c + b {
        add_left_map_is_order_embedding(c)
        order_embedding_reflects_gt(add_left_map(c), a, b)
        a > b
    }
}

/// A strict reverse comparison after adding on the right reflects to the original comparison.
theorem gt_of_add_gt_add_right[G: AddOrderedGroup](a: G, b: G, c: G) {
    a + c > b + c implies a > b
} by {
    if a + c > b + c {
        add_right_map_is_order_embedding(c)
        order_embedding_reflects_gt(add_right_map(c), a, b)
        a > b
    }
}

/// Adding a fixed element on the left preserves and reflects a non-strict comparison.
theorem add_le_add_left_iff[G: AddLeftOrderedGroup](a: G, b: G, c: G) {
    c + a <= c + b = (a <= b)
} by {
    let e = add_left_order_iso(c)
    add_left_order_iso_map(c)
    order_iso_le_iff_le(e, a, b)
}

/// Adding a fixed element on the right preserves and reflects a non-strict comparison.
theorem add_le_add_right_iff[G: AddOrderedGroup](a: G, b: G, c: G) {
    a + c <= b + c = (a <= b)
} by {
    let e = add_right_order_iso(c)
    add_right_order_iso_map(c)
    order_iso_le_iff_le(e, a, b)
}

theorem add_le_iff_le_sub_right[G: AddOrderedGroup](a: G, b: G, c: G) {
    a + b <= c = (a <= c - b)
} by {
    add_le_add_right_iff(a + b, c, -b)
    (a + b) + -b <= c + -b = (a + b <= c)
    (a + b) + -b = a + (b + -b)
    b + -b = G.0
    a + G.0 = a
    c + -b = c - b
    a <= c - b = (a + b <= c)
    a + b <= c = (a <= c - b)
}

/// Adding a fixed element on the left preserves and reflects a strict comparison.
theorem add_lt_add_left_iff[G: AddLeftOrderedGroup](a: G, b: G, c: G) {
    c + a < c + b = (a < b)
} by {
    let e = add_left_order_iso(c)
    add_left_order_iso_map(c)
    order_iso_lt_iff_lt(e, a, b)
}

/// Adding a fixed element on the right preserves and reflects a strict comparison.
theorem add_lt_add_right_iff[G: AddOrderedGroup](a: G, b: G, c: G) {
    a + c < b + c = (a < b)
} by {
    let e = add_right_order_iso(c)
    add_right_order_iso_map(c)
    order_iso_lt_iff_lt(e, a, b)
}

/// Adding a fixed element on the left preserves and reflects a reverse non-strict comparison.
theorem add_ge_add_left_iff[G: AddLeftOrderedGroup](a: G, b: G, c: G) {
    c + a >= c + b = (a >= b)
} by {
    let e = add_left_order_iso(c)
    add_left_order_iso_map(c)
    order_iso_ge_iff_ge(e, a, b)
}

/// Adding a fixed element on the right preserves and reflects a reverse non-strict comparison.
theorem add_ge_add_right_iff[G: AddOrderedGroup](a: G, b: G, c: G) {
    a + c >= b + c = (a >= b)
} by {
    let e = add_right_order_iso(c)
    add_right_order_iso_map(c)
    order_iso_ge_iff_ge(e, a, b)
}

/// Adding a fixed element on the left preserves and reflects a strict reverse comparison.
theorem add_gt_add_left_iff[G: AddLeftOrderedGroup](a: G, b: G, c: G) {
    c + a > c + b = (a > b)
} by {
    let e = add_left_order_iso(c)
    add_left_order_iso_map(c)
    order_iso_gt_iff_gt(e, a, b)
}

/// Adding a fixed element on the right preserves and reflects a strict reverse comparison.
theorem add_gt_add_right_iff[G: AddOrderedGroup](a: G, b: G, c: G) {
    a + c > b + c = (a > b)
} by {
    let e = add_right_order_iso(c)
    add_right_order_iso_map(c)
    order_iso_gt_iff_gt(e, a, b)
}

/// Two non-strict comparisons add componentwise.
theorem add_le_add[G: AddOrderedGroup](a: G, b: G, c: G, d: G) {
    a <= b and c <= d implies a + c <= b + d
} by {
    if a <= b and c <= d {
        add_inequality(a, b, c, d)
        a + c <= b + d
    }
}

/// Two reverse non-strict comparisons add componentwise.
theorem add_ge_add[G: AddOrderedGroup](a: G, b: G, c: G, d: G) {
    a >= b and c >= d implies a + c >= b + d
} by {
    if a >= b and c >= d {
        d <= c
        add_le_add(b, a, d, c)
        b + d <= a + c
        a + c >= b + d
    }
}

/// The map obtained by taking additive inverses.
define neg_map[G: AddOrderedGroup](x: G) -> G {
    -x
}

/// The inverse map is negation.
theorem neg_map_apply[G: AddOrderedGroup](x: G) {
    neg_map(x) = -x
}

/// Negation reverses the non-strict order.
theorem neg_map_is_antitone[G: AddOrderedGroup] {
    is_antitone(neg_map[G])
} by {
    forall(x: G, y: G) {
        if x <= y {
            minus_flips_inequality(x, y)
            neg_map(y) <= neg_map(x)
        }
    }
}

/// Negation reflects the reversed non-strict order.
theorem neg_map_reflects_le[G: AddOrderedGroup](x: G, y: G) {
    neg_map(y) <= neg_map(x) implies x <= y
} by {
    if neg_map(y) <= neg_map(x) {
        minus_flips_inequality(-y, -x)
        --x <= --y
        --x = x
        --y = y
        x <= y
    }
}

/// Negation reverses strict order.
theorem neg_map_is_strict_antitone[G: AddOrderedGroup] {
    is_strict_antitone(neg_map[G])
} by {
    forall(x: G, y: G) {
        if x < y {
            minus_flips_inequality(x, y)
            if -y = -x {
                x != y
                false
            }
            -y < -x
            neg_map(y) < neg_map(x)
        }
    }
}

/// Negation as an order-dual isomorphism.
let neg_order_dual_iso[G: AddOrderedGroup](anchor: G) -> result: OrderDualIso[G, G] satisfy {
    OrderDualIso[G, G].new(neg_map[G], neg_map[G]) = Option.some(result)
} by {
    forall(x: G) {
        --x = x
        neg_map(neg_map(x)) = x
    }
    neg_map_is_antitone[G]
    is_order_dual_iso_pair(neg_map[G], neg_map[G])
}

/// The map of the negation order-dual isomorphism is negation.
theorem neg_order_dual_iso_map[G: AddOrderedGroup](anchor: G) {
    neg_order_dual_iso(anchor).map = neg_map[G]
} by {
    let e = neg_order_dual_iso(anchor)
    OrderDualIso[G, G].new(neg_map[G], neg_map[G]) = Option.some(e)
    order_dual_iso_new_map(neg_map[G], neg_map[G], e)
}

/// The inverse of the negation order-dual isomorphism is negation.
theorem neg_order_dual_iso_inv[G: AddOrderedGroup](anchor: G) {
    neg_order_dual_iso(anchor).inv = neg_map[G]
} by {
    let e = neg_order_dual_iso(anchor)
    OrderDualIso[G, G].new(neg_map[G], neg_map[G]) = Option.some(e)
    order_dual_iso_new_inv(neg_map[G], neg_map[G], e)
}

/// Negation reflects the reversed strict order.
theorem neg_map_reflects_lt[G: AddOrderedGroup](x: G, y: G) {
    neg_map(y) < neg_map(x) implies x < y
} by {
    if neg_map(y) < neg_map(x) {
        neg_map(y) = -y
        neg_map(x) = -x
        neg_map_is_strict_antitone[G]
        is_strict_antitone(neg_map[G])
        neg_map(-x) < neg_map(-y)
        --x < --y
        x < y
    }
}

/// Negation turns a non-strict comparison into the reversed non-strict comparison.
theorem neg_le_neg[G: AddOrderedGroup](a: G, b: G) {
    a <= b implies -b <= -a
} by {
    if a <= b {
        minus_flips_inequality(a, b)
        -b <= -a
    }
}

/// A non-strict comparison between negatives reflects to the reversed comparison.
theorem le_of_neg_le_neg[G: AddOrderedGroup](a: G, b: G) {
    -b <= -a implies a <= b
} by {
    if -b <= -a {
        neg_map_reflects_le(a, b)
        a <= b
    }
}

/// Negation turns a strict comparison into the reversed strict comparison.
theorem neg_lt_neg[G: AddOrderedGroup](a: G, b: G) {
    a < b implies -b < -a
} by {
    if a < b {
        neg_map_is_strict_antitone[G]
        is_strict_antitone(neg_map[G])
        neg_map(b) < neg_map(a)
        -b < -a
    }
}

/// A strict comparison between negatives reflects to the reversed comparison.
theorem lt_of_neg_lt_neg[G: AddOrderedGroup](a: G, b: G) {
    -b < -a implies a < b
} by {
    if -b < -a {
        neg_map_reflects_lt(a, b)
        a < b
    }
}

/// Negation turns a reverse non-strict comparison into a non-strict comparison.
theorem neg_ge_neg[G: AddOrderedGroup](a: G, b: G) {
    a >= b implies -b >= -a
} by {
    if a >= b {
        neg_le_neg(b, a)
        -a <= -b
        -b >= -a
    }
}

/// A reverse non-strict comparison between negatives reflects to the original reverse comparison.
theorem ge_of_neg_ge_neg[G: AddOrderedGroup](a: G, b: G) {
    -b >= -a implies a >= b
} by {
    if -b >= -a {
        le_of_neg_le_neg(b, a)
        b <= a
        a >= b
    }
}

/// Negation turns a strict reverse comparison into a strict comparison.
theorem neg_gt_neg[G: AddOrderedGroup](a: G, b: G) {
    a > b implies -b > -a
} by {
    if a > b {
        neg_lt_neg(b, a)
        -a < -b
        -b > -a
    }
}

/// A strict reverse comparison between negatives reflects to the original strict reverse comparison.
theorem gt_of_neg_gt_neg[G: AddOrderedGroup](a: G, b: G) {
    -b > -a implies a > b
} by {
    if -b > -a {
        lt_of_neg_lt_neg(b, a)
        b < a
        a > b
    }
}

/// Negation turns a non-strict comparison exactly into the reversed comparison.
theorem neg_le_neg_iff[G: AddOrderedGroup](a: G, b: G) {
    -b <= -a = (a <= b)
} by {
    if -b <= -a {
        le_of_neg_le_neg(a, b)
        a <= b
    }
    if a <= b {
        neg_le_neg(a, b)
        -b <= -a
    }
}

/// Negation turns a strict comparison exactly into the reversed comparison.
theorem neg_lt_neg_iff[G: AddOrderedGroup](a: G, b: G) {
    -b < -a = (a < b)
} by {
    if -b < -a {
        lt_of_neg_lt_neg(a, b)
        a < b
    }
    if a < b {
        neg_lt_neg(a, b)
        -b < -a
    }
}

/// Negation turns a reverse non-strict comparison exactly into a non-strict comparison.
theorem neg_ge_neg_iff[G: AddOrderedGroup](a: G, b: G) {
    -b >= -a = (a >= b)
} by {
    if -b >= -a {
        ge_of_neg_ge_neg(a, b)
        a >= b
    }
    if a >= b {
        neg_ge_neg(a, b)
        -b >= -a
    }
}

/// Negation turns a strict reverse comparison exactly into a strict comparison.
theorem neg_gt_neg_iff[G: AddOrderedGroup](a: G, b: G) {
    -b > -a = (a > b)
} by {
    if -b > -a {
        gt_of_neg_gt_neg(a, b)
        a > b
    }
    if a > b {
        neg_gt_neg(a, b)
        -b > -a
    }
}
