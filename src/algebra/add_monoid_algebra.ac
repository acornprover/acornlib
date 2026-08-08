/// Additive monoid algebras as finitely supported coefficient functions.

from finite_set import FiniteSet
from algebra.module.finite_support import has_support_in, has_support_in_apply_zero, is_finitely_supported,
    pointwise_add_has_support_in_union, pointwise_neg_has_support_in,
    pointwise_neg_is_finitely_supported
from data.basic.function_algebra import pointwise_add, pointwise_zero, pointwise_neg
from data.basic.functions import function_extensionality
from algebra.add import Add
from algebra.zero import Zero
from algebra.ring.ring import Ring
from semiring import Semiring
from data.basic.set import Set, empty_set_contains_eq, singleton_contains_eq, singleton_set_is_finite

/// The additive monoid algebra over a semiring `R` with basis indexed by `M`.
/// Its elements are finitely supported coefficient functions `M -> R`.
structure AddMonoidAlgebra[R: Semiring, M] {
    /// The coefficient function.
    coeff: M -> R
} constraint {
    is_finitely_supported(coeff)
}

/// Additive monoid algebra extensionality from equality of coefficient functions.
theorem add_monoid_algebra_ext[R: Semiring, M](p: AddMonoidAlgebra[R, M], q: AddMonoidAlgebra[R, M]) {
    p.coeff = q.coeff implies p = q
} by {
    p.coeff = q.coeff
}

/// Additive monoid algebra extensionality from pointwise equality of coefficients.
theorem add_monoid_algebra_ext_pointwise[R: Semiring, M](
    p: AddMonoidAlgebra[R, M], q: AddMonoidAlgebra[R, M]
) {
    (forall(m: M) { p.coeff(m) = q.coeff(m) }) implies p = q
} by {
    function_extensionality(p.coeff, q.coeff)
    add_monoid_algebra_ext(p, q)
}

/// Equal additive monoid algebra elements have equal coefficient functions.
theorem add_monoid_algebra_eq_coeff[R: Semiring, M](p: AddMonoidAlgebra[R, M], q: AddMonoidAlgebra[R, M]) {
    p = q implies p.coeff = q.coeff
}

/// Equal additive monoid algebra elements have equal coefficients at every index.
theorem add_monoid_algebra_eq_coeff_at[R: Semiring, M](
    p: AddMonoidAlgebra[R, M], q: AddMonoidAlgebra[R, M], m: M
) {
    p = q implies p.coeff(m) = q.coeff(m)
} by {
}

/// Equality of additive monoid algebra elements is pointwise equality of coefficients.
theorem add_monoid_algebra_eq_iff_coeff_eq[R: Semiring, M](
    p: AddMonoidAlgebra[R, M], q: AddMonoidAlgebra[R, M]
) {
    p = q = forall(m: M) { p.coeff(m) = q.coeff(m) }
} by {
    if p = q {
        forall(m: M) {
            add_monoid_algebra_eq_coeff_at(p, q, m)
            p.coeff(m) = q.coeff(m)
        }
    }
    if forall(m: M) { p.coeff(m) = q.coeff(m) } {
        add_monoid_algebra_ext_pointwise(p, q)
        p = q
    }
}

/// The singleton finite set at a basis index.
let add_monoid_algebra_singleton_support[M](m: M) -> result: FiniteSet[M] satisfy {
    FiniteSet[M].new(Set[M].singleton(m)) = Option.some(result)
} by {
    singleton_set_is_finite(m)
}

/// The coefficient function of a single basis term.
define add_monoid_algebra_single_coeff[R: Semiring, M](m: M, r: R, n: M) -> R {
    if n = m {
        r
    } else {
        R.0
    }
}

/// A single basis coefficient function is zero outside its singleton support.
theorem add_monoid_algebra_single_coeff_zero_of_not_contains[R: Semiring, M](m: M, r: R, n: M) {
    not add_monoid_algebra_singleton_support(m).contains(n) implies
    add_monoid_algebra_single_coeff[R, M](m, r, n) = R.0
} by {
    add_monoid_algebra_singleton_support(m).underlying_set = Set[M].singleton(m)
    singleton_contains_eq(m, n)
    not n = m
}

/// A single basis coefficient function is supported at its basis index.
theorem add_monoid_algebra_single_coeff_has_support[R: Semiring, M](m: M, r: R) {
    has_support_in(
        add_monoid_algebra_single_coeff[R, M](m, r),
        add_monoid_algebra_singleton_support(m)
    )
} by {
    forall(n: M) {
        add_monoid_algebra_single_coeff_zero_of_not_contains[R, M](m, r, n)
    }
}

/// A single basis coefficient function is finitely supported.
theorem add_monoid_algebra_single_coeff_is_finitely_supported[R: Semiring, M](m: M, r: R) {
    is_finitely_supported(add_monoid_algebra_single_coeff[R, M](m, r))
} by {
    add_monoid_algebra_single_coeff_has_support[R, M](m, r)
}

/// The additive monoid algebra element with one nonzero basis coefficient.
let add_monoid_algebra_single[R: Semiring, M](m: M, r: R) -> result: AddMonoidAlgebra[R, M] satisfy {
    AddMonoidAlgebra[R, M].new(add_monoid_algebra_single_coeff[R, M](m, r)) = Option.some(result)
} by {
    add_monoid_algebra_single_coeff_is_finitely_supported[R, M](m, r)
}

/// The single basis element has the chosen coefficient at its basis index.
theorem add_monoid_algebra_single_coeff_self[R: Semiring, M](m: M, r: R) {
    add_monoid_algebra_single[R, M](m, r).coeff(m) = r
} by {
    add_monoid_algebra_single[R, M](m, r).coeff = add_monoid_algebra_single_coeff[R, M](m, r)
    add_monoid_algebra_single_coeff[R, M](m, r, m) = r
}

/// The single basis element is zero away from its basis index.
theorem add_monoid_algebra_single_coeff_of_ne[R: Semiring, M](m: M, r: R, n: M) {
    n != m implies add_monoid_algebra_single[R, M](m, r).coeff(n) = R.0
} by {
    add_monoid_algebra_single[R, M](m, r).coeff = add_monoid_algebra_single_coeff[R, M](m, r)
    add_monoid_algebra_single_coeff[R, M](m, r, n) = R.0
}

/// The zero element of an additive monoid algebra.
let add_monoid_algebra_zero[R: Semiring, M]: AddMonoidAlgebra[R, M] satisfy {
    AddMonoidAlgebra[R, M].new(pointwise_zero[M, R]) = Option.some(add_monoid_algebra_zero)
}

/// The sum of two additive monoid algebra elements.
let add_monoid_algebra_add[R: Semiring, M](p: AddMonoidAlgebra[R, M], q: AddMonoidAlgebra[R, M]) -> result: AddMonoidAlgebra[R, M] satisfy {
    AddMonoidAlgebra[R, M].new(pointwise_add(p.coeff, q.coeff)) = Option.some(result)
} by {
    let s: FiniteSet[M] satisfy {
        has_support_in(p.coeff, s)
    }
    let u: FiniteSet[M] satisfy {
        has_support_in(q.coeff, u)
    }
    pointwise_add_has_support_in_union[M, R](p.coeff, q.coeff, s, u)
    exists(v: FiniteSet[M]) {
        v = s.union(u) and has_support_in(pointwise_add(p.coeff, q.coeff), v)
    }
}

attributes AddMonoidAlgebra[R: Semiring, M] {
    /// Additive monoid algebra extensionality from equality of coefficients.
    let ext = add_monoid_algebra_ext[R, M]

    /// The single basis term with a chosen coefficient.
    let single: (M, R) -> AddMonoidAlgebra[R, M] = add_monoid_algebra_single

    /// The zero element of an additive monoid algebra.
    let zero: AddMonoidAlgebra[R, M] = add_monoid_algebra_zero[R, M]

    /// The coefficientwise sum of two additive monoid algebra elements.
    define add(self, other: AddMonoidAlgebra[R, M]) -> AddMonoidAlgebra[R, M] {
        add_monoid_algebra_add(self, other)
    }
}

/// Additive monoid algebra elements have coefficientwise addition.
instance AddMonoidAlgebra[R: Semiring, M]: Add {
    let add = AddMonoidAlgebra[R, M].add
}

/// Additive monoid algebra elements have a zero element.
instance AddMonoidAlgebra[R: Semiring, M]: Zero {
    let 0 = AddMonoidAlgebra[R, M].zero
}

/// The zero element has zero coefficient at every index.
theorem add_monoid_algebra_zero_coeff[R: Semiring, M](m: M) {
    AddMonoidAlgebra[R, M].zero.coeff(m) = R.0
} by {
    add_monoid_algebra_zero[R, M].coeff = pointwise_zero[M, R]
    pointwise_zero[M, R](m) = R.0
}

/// Addition of additive monoid algebra elements adds coefficients pointwise.
theorem add_monoid_algebra_add_coeff[R: Semiring, M](p: AddMonoidAlgebra[R, M], q: AddMonoidAlgebra[R, M], m: M) {
    (p + q).coeff(m) = p.coeff(m) + q.coeff(m)
} by {
    add_monoid_algebra_add(p, q).coeff = pointwise_add(p.coeff, q.coeff)
}

/// Addition in an additive monoid algebra is associative.
theorem add_monoid_algebra_add_assoc[R: Semiring, M](
    p: AddMonoidAlgebra[R, M], q: AddMonoidAlgebra[R, M], r: AddMonoidAlgebra[R, M]
) {
    p + (q + r) = (p + q) + r
} by {
    forall(m: M) {
        add_monoid_algebra_add_coeff(q, r, m)
        add_monoid_algebra_add_coeff(p, add_monoid_algebra_add(q, r), m)
        add_monoid_algebra_add_coeff(p, q, m)
        add_monoid_algebra_add_coeff(add_monoid_algebra_add(p, q), r, m)
        p.coeff(m) + (q.coeff(m) + r.coeff(m)) = (p.coeff(m) + q.coeff(m)) + r.coeff(m)
        add_monoid_algebra_add(p, add_monoid_algebra_add(q, r)).coeff(m) =
            add_monoid_algebra_add(add_monoid_algebra_add(p, q), r).coeff(m)
    }
    add_monoid_algebra_ext_pointwise(
        add_monoid_algebra_add(p, add_monoid_algebra_add(q, r)),
        add_monoid_algebra_add(add_monoid_algebra_add(p, q), r)
    )
}

/// Addition in an additive monoid algebra is commutative.
theorem add_monoid_algebra_add_comm[R: Semiring, M](p: AddMonoidAlgebra[R, M], q: AddMonoidAlgebra[R, M]) {
    p + q = q + p
} by {
    forall(m: M) {
        add_monoid_algebra_add_coeff(p, q, m)
        add_monoid_algebra_add_coeff(q, p, m)
        p.coeff(m) + q.coeff(m) = q.coeff(m) + p.coeff(m)
        add_monoid_algebra_add(p, q).coeff(m) = add_monoid_algebra_add(q, p).coeff(m)
    }
    add_monoid_algebra_ext_pointwise(add_monoid_algebra_add(p, q), add_monoid_algebra_add(q, p))
}

/// Adding zero on the right changes no additive monoid algebra element.
theorem add_monoid_algebra_add_zero_right[R: Semiring, M](p: AddMonoidAlgebra[R, M]) {
    p + AddMonoidAlgebra[R, M].zero = p
} by {
    forall(m: M) {
        add_monoid_algebra_add_coeff(p, add_monoid_algebra_zero[R, M], m)
        add_monoid_algebra_zero_coeff[R, M](m)
        add_monoid_algebra_add(p, add_monoid_algebra_zero[R, M]).coeff(m) = p.coeff(m)
    }
    add_monoid_algebra_ext_pointwise(add_monoid_algebra_add(p, add_monoid_algebra_zero[R, M]), p)
}

/// Adding zero on the left changes no additive monoid algebra element.
theorem add_monoid_algebra_add_zero_left[R: Semiring, M](p: AddMonoidAlgebra[R, M]) {
    AddMonoidAlgebra[R, M].zero + p = p
} by {
    forall(m: M) {
        add_monoid_algebra_add_coeff(add_monoid_algebra_zero[R, M], p, m)
        add_monoid_algebra_zero_coeff[R, M](m)
        add_monoid_algebra_add(add_monoid_algebra_zero[R, M], p).coeff(m) = p.coeff(m)
    }
    add_monoid_algebra_ext_pointwise(add_monoid_algebra_add(add_monoid_algebra_zero[R, M], p), p)
}

/// The zero additive monoid algebra element is supported in every finite set.
theorem add_monoid_algebra_zero_has_support_in[R: Semiring, M](s: FiniteSet[M]) {
    has_support_in(AddMonoidAlgebra[R, M].zero.coeff, s)
} by {
    forall(m: M) {
        add_monoid_algebra_zero_coeff[R, M](m)
    }
}

/// The zero additive monoid algebra element is supported in the empty finite set.
theorem add_monoid_algebra_zero_has_support_empty[R: Semiring, M] {
    has_support_in(AddMonoidAlgebra[R, M].zero.coeff, FiniteSet[M].empty)
} by {
    add_monoid_algebra_zero_has_support_in[R, M](FiniteSet[M].empty)
}

/// An additive monoid algebra element supported in the empty finite set is zero.
theorem add_monoid_algebra_eq_zero_of_has_support_empty[R: Semiring, M](p: AddMonoidAlgebra[R, M]) {
    has_support_in(p.coeff, FiniteSet[M].empty) implies p = AddMonoidAlgebra[R, M].zero
} by {
    forall(m: M) {
        empty_set_contains_eq[M](m)
        not FiniteSet[M].empty.contains(m)
        has_support_in_apply_zero[M, R](p.coeff, FiniteSet[M].empty, m)
        add_monoid_algebra_zero_coeff[R, M](m)
        p.coeff(m) = AddMonoidAlgebra[R, M].zero.coeff(m)
    }
    add_monoid_algebra_ext_pointwise(p, AddMonoidAlgebra[R, M].zero)
}

/// A single basis term is supported in its singleton support.
theorem add_monoid_algebra_single_has_support_in_singleton[R: Semiring, M](m: M, r: R) {
    has_support_in(AddMonoidAlgebra[R, M].single(m, r).coeff, add_monoid_algebra_singleton_support(m))
} by {
    AddMonoidAlgebra[R, M].new(add_monoid_algebra_single_coeff[R, M](m, r)) =
        Option.some(AddMonoidAlgebra[R, M].single(m, r))
    AddMonoidAlgebra[R, M].single(m, r).coeff = add_monoid_algebra_single_coeff[R, M](m, r)
    add_monoid_algebra_single_coeff_has_support[R, M](m, r)
}

/// The sum of two additive monoid algebra elements is supported in the union of supports.
theorem add_monoid_algebra_add_has_support_in_union[R: Semiring, M](
    p: AddMonoidAlgebra[R, M],
    q: AddMonoidAlgebra[R, M],
    s: FiniteSet[M],
    u: FiniteSet[M]
) {
    has_support_in(p.coeff, s) and has_support_in(q.coeff, u) implies
    has_support_in((p + q).coeff, s.union(u))
} by {
    (p + q).coeff = pointwise_add(p.coeff, q.coeff)
    pointwise_add_has_support_in_union[M, R](p.coeff, q.coeff, s, u)
}

/// The coefficient function obtained by scalar multiplication.
define add_monoid_algebra_smul_coeff_fn[R: Semiring, M](r: R, p: AddMonoidAlgebra[R, M], m: M) -> R {
    r * p.coeff(m)
}

/// Scalar multiplication is zero outside any support of the original element.
theorem add_monoid_algebra_smul_coeff_zero_of_not_contains[R: Semiring, M](
    r: R,
    p: AddMonoidAlgebra[R, M],
    s: FiniteSet[M],
    m: M
) {
    has_support_in(p.coeff, s) and not s.contains(m) implies
    add_monoid_algebra_smul_coeff_fn(r, p, m) = R.0
} by {
    has_support_in_apply_zero[M, R](p.coeff, s, m)
    add_monoid_algebra_smul_coeff_fn(r, p, m) = r * R.0
}

/// Scalar multiplication preserves a specified finite support.
theorem add_monoid_algebra_smul_coeff_has_support[R: Semiring, M](
    r: R,
    p: AddMonoidAlgebra[R, M],
    s: FiniteSet[M]
) {
    has_support_in(p.coeff, s) implies has_support_in(add_monoid_algebra_smul_coeff_fn(r, p), s)
} by {
    forall(m: M) {
        add_monoid_algebra_smul_coeff_zero_of_not_contains(r, p, s, m)
    }
}

/// Scalar multiplication of an additive monoid algebra element has finite support.
theorem add_monoid_algebra_smul_coeff_is_finitely_supported[R: Semiring, M](r: R, p: AddMonoidAlgebra[R, M]) {
    is_finitely_supported(add_monoid_algebra_smul_coeff_fn(r, p))
} by {
    let s: FiniteSet[M] satisfy {
        has_support_in(p.coeff, s)
    }
    add_monoid_algebra_smul_coeff_has_support(r, p, s)
}

/// Scalar multiplication of an additive monoid algebra element.
let add_monoid_algebra_smul[R: Semiring, M](r: R, p: AddMonoidAlgebra[R, M]) -> result: AddMonoidAlgebra[R, M] satisfy {
    AddMonoidAlgebra[R, M].new(add_monoid_algebra_smul_coeff_fn(r, p)) = Option.some(result)
} by {
    add_monoid_algebra_smul_coeff_is_finitely_supported(r, p)
}

attributes AddMonoidAlgebra[R: Semiring, M] {
    /// Scalar multiplication by a coefficient.
    define smul(self, r: R) -> AddMonoidAlgebra[R, M] {
        add_monoid_algebra_smul(r, self)
    }
}

/// Scalar multiplication multiplies every coefficient by the scalar.
theorem add_monoid_algebra_smul_coeff[R: Semiring, M](r: R, p: AddMonoidAlgebra[R, M], m: M) {
    p.smul(r).coeff(m) = r * p.coeff(m)
} by {
    add_monoid_algebra_smul(r, p).coeff = add_monoid_algebra_smul_coeff_fn(r, p)
}

/// Scalar multiplication preserves a specified support of an additive monoid algebra element.
theorem add_monoid_algebra_smul_has_support_in[R: Semiring, M](
    r: R,
    p: AddMonoidAlgebra[R, M],
    s: FiniteSet[M]
) {
    has_support_in(p.coeff, s) implies has_support_in(p.smul(r).coeff, s)
} by {
    p.smul(r).coeff = add_monoid_algebra_smul_coeff_fn(r, p)
    add_monoid_algebra_smul_coeff_has_support(r, p, s)
}

/// Scalar multiplication of zero gives zero.
theorem add_monoid_algebra_smul_zero[R: Semiring, M](r: R) {
    AddMonoidAlgebra[R, M].zero.smul(r) = AddMonoidAlgebra[R, M].zero
} by {
    forall(m: M) {
        add_monoid_algebra_smul_coeff(r, AddMonoidAlgebra[R, M].zero, m)
        add_monoid_algebra_zero_coeff[R, M](m)
        r * R.0 = R.0
        AddMonoidAlgebra[R, M].zero.smul(r).coeff(m) = AddMonoidAlgebra[R, M].zero.coeff(m)
    }
    add_monoid_algebra_ext_pointwise(AddMonoidAlgebra[R, M].zero.smul(r), AddMonoidAlgebra[R, M].zero)
}

/// The zero scalar sends every additive monoid algebra element to zero.
theorem add_monoid_algebra_zero_smul[R: Semiring, M](p: AddMonoidAlgebra[R, M]) {
    p.smul(R.0) = AddMonoidAlgebra[R, M].zero
} by {
    forall(m: M) {
        add_monoid_algebra_smul_coeff(R.0, p, m)
        add_monoid_algebra_zero_coeff[R, M](m)
        p.smul(R.0).coeff(m) = AddMonoidAlgebra[R, M].zero.coeff(m)
    }
    add_monoid_algebra_ext_pointwise(p.smul(R.0), AddMonoidAlgebra[R, M].zero)
}

/// The identity scalar changes no additive monoid algebra element.
theorem add_monoid_algebra_one_smul[R: Semiring, M](p: AddMonoidAlgebra[R, M]) {
    p.smul(R.1) = p
} by {
    forall(m: M) {
        add_monoid_algebra_smul_coeff(R.1, p, m)
        p.smul(R.1).coeff(m) = p.coeff(m)
    }
    add_monoid_algebra_ext_pointwise(p.smul(R.1), p)
}

/// Scalar multiplication distributes over additive monoid algebra addition.
theorem add_monoid_algebra_smul_add[R: Semiring, M](r: R, p: AddMonoidAlgebra[R, M], q: AddMonoidAlgebra[R, M]) {
    (p + q).smul(r) = p.smul(r) + q.smul(r)
} by {
    forall(m: M) {
        add_monoid_algebra_smul_coeff(r, p + q, m)
        add_monoid_algebra_add_coeff(p, q, m)
        add_monoid_algebra_add_coeff(p.smul(r), q.smul(r), m)
        add_monoid_algebra_smul_coeff(r, p, m)
        add_monoid_algebra_smul_coeff(r, q, m)
        r * (p.coeff(m) + q.coeff(m)) = r * p.coeff(m) + r * q.coeff(m)
        (p + q).smul(r).coeff(m) = (p.smul(r) + q.smul(r)).coeff(m)
    }
    add_monoid_algebra_ext_pointwise((p + q).smul(r), p.smul(r) + q.smul(r))
}

/// Scalar multiplication distributes over addition of scalars.
theorem add_monoid_algebra_add_smul[R: Semiring, M](r: R, s: R, p: AddMonoidAlgebra[R, M]) {
    p.smul(r + s) = p.smul(r) + p.smul(s)
} by {
    forall(m: M) {
        add_monoid_algebra_smul_coeff(r + s, p, m)
        add_monoid_algebra_add_coeff(p.smul(r), p.smul(s), m)
        add_monoid_algebra_smul_coeff(r, p, m)
        add_monoid_algebra_smul_coeff(s, p, m)
        p.smul(r + s).coeff(m) = (p.smul(r) + p.smul(s)).coeff(m)
    }
    add_monoid_algebra_ext_pointwise(p.smul(r + s), p.smul(r) + p.smul(s))
}

/// Scalar multiplication is compatible with multiplication of scalars.
theorem add_monoid_algebra_smul_assoc[R: Semiring, M](r: R, s: R, p: AddMonoidAlgebra[R, M]) {
    p.smul(r * s) = p.smul(s).smul(r)
} by {
    forall(m: M) {
        add_monoid_algebra_smul_coeff(r * s, p, m)
        add_monoid_algebra_smul_coeff(r, p.smul(s), m)
        add_monoid_algebra_smul_coeff(s, p, m)
        p.smul(r * s).coeff(m) = p.smul(s).smul(r).coeff(m)
    }
    add_monoid_algebra_ext_pointwise(p.smul(r * s), p.smul(s).smul(r))
}

/// The coefficientwise additive inverse of an additive monoid algebra element.
let add_monoid_algebra_neg[R: Ring, M](p: AddMonoidAlgebra[R, M]) -> result: AddMonoidAlgebra[R, M] satisfy {
    AddMonoidAlgebra[R, M].new(pointwise_neg(p.coeff)) = Option.some(result)
} by {
    pointwise_neg_is_finitely_supported[M, R](p.coeff)
}

attributes AddMonoidAlgebra[R: Ring, M] {
    /// The coefficientwise additive inverse of an additive monoid algebra element.
    define neg(self) -> AddMonoidAlgebra[R, M] {
        add_monoid_algebra_neg(self)
    }

    /// The coefficientwise difference of two additive monoid algebra elements.
    define sub(self, other: AddMonoidAlgebra[R, M]) -> AddMonoidAlgebra[R, M] {
        self + other.neg
    }
}

/// Negation of an additive monoid algebra element negates every coefficient.
theorem add_monoid_algebra_neg_coeff[R: Ring, M](p: AddMonoidAlgebra[R, M], m: M) {
    p.neg.coeff(m) = -p.coeff(m)
} by {
    add_monoid_algebra_neg(p).coeff = pointwise_neg(p.coeff)
}

/// The coefficientwise difference subtracts coefficients.
theorem add_monoid_algebra_sub_coeff[R: Ring, M](p: AddMonoidAlgebra[R, M], q: AddMonoidAlgebra[R, M], m: M) {
    p.sub(q).coeff(m) = p.coeff(m) + -q.coeff(m)
} by {
    add_monoid_algebra_add_coeff(p, q.neg, m)
    add_monoid_algebra_neg_coeff(q, m)
}

/// Negation preserves a specified support of an additive monoid algebra element.
theorem add_monoid_algebra_neg_has_support_in[R: Ring, M](p: AddMonoidAlgebra[R, M], s: FiniteSet[M]) {
    has_support_in(p.coeff, s) implies has_support_in(p.neg.coeff, s)
} by {
    p.neg.coeff = pointwise_neg(p.coeff)
    pointwise_neg_has_support_in[M, R](p.coeff, s)
}

/// Subtraction is supported in the union of supports.
theorem add_monoid_algebra_sub_has_support_in_union[R: Ring, M](
    p: AddMonoidAlgebra[R, M],
    q: AddMonoidAlgebra[R, M],
    s: FiniteSet[M],
    u: FiniteSet[M]
) {
    has_support_in(p.coeff, s) and has_support_in(q.coeff, u) implies
    has_support_in(p.sub(q).coeff, s.union(u))
} by {
    add_monoid_algebra_neg_has_support_in(q, u)
    add_monoid_algebra_add_has_support_in_union(p, q.neg, s, u)
}

/// Adding the additive inverse on the right gives zero.
theorem add_monoid_algebra_add_neg_right[R: Ring, M](p: AddMonoidAlgebra[R, M]) {
    p + p.neg = AddMonoidAlgebra[R, M].zero
} by {
    forall(m: M) {
        add_monoid_algebra_add_coeff(p, p.neg, m)
        add_monoid_algebra_neg_coeff(p, m)
        p.coeff(m) + -p.coeff(m) = R.0
        add_monoid_algebra_zero_coeff[R, M](m)
        (p + p.neg).coeff(m) = AddMonoidAlgebra[R, M].zero.coeff(m)
    }
    add_monoid_algebra_ext_pointwise(p + p.neg, AddMonoidAlgebra[R, M].zero)
}

/// Adding the additive inverse on the left gives zero.
theorem add_monoid_algebra_add_neg_left[R: Ring, M](p: AddMonoidAlgebra[R, M]) {
    p.neg + p = AddMonoidAlgebra[R, M].zero
} by {
    forall(m: M) {
        add_monoid_algebra_add_coeff(p.neg, p, m)
        add_monoid_algebra_neg_coeff(p, m)
        add_monoid_algebra_zero_coeff[R, M](m)
        (p.neg + p).coeff(m) = AddMonoidAlgebra[R, M].zero.coeff(m)
    }
    add_monoid_algebra_ext_pointwise(p.neg + p, AddMonoidAlgebra[R, M].zero)
}

/// Negating zero in an additive monoid algebra gives zero.
theorem add_monoid_algebra_neg_zero[R: Ring, M] {
    AddMonoidAlgebra[R, M].zero.neg = AddMonoidAlgebra[R, M].zero
} by {
    forall(m: M) {
        add_monoid_algebra_neg_coeff(AddMonoidAlgebra[R, M].zero, m)
        add_monoid_algebra_zero_coeff[R, M](m)
        AddMonoidAlgebra[R, M].zero.neg.coeff(m) = AddMonoidAlgebra[R, M].zero.coeff(m)
    }
    add_monoid_algebra_ext_pointwise(AddMonoidAlgebra[R, M].zero.neg, AddMonoidAlgebra[R, M].zero)
}

/// Negation distributes over addition in an additive monoid algebra.
theorem add_monoid_algebra_neg_add[R: Ring, M](p: AddMonoidAlgebra[R, M], q: AddMonoidAlgebra[R, M]) {
    (p + q).neg = p.neg + q.neg
} by {
    forall(m: M) {
        add_monoid_algebra_neg_coeff(p + q, m)
        add_monoid_algebra_add_coeff(p, q, m)
        add_monoid_algebra_add_coeff(p.neg, q.neg, m)
        add_monoid_algebra_neg_coeff(p, m)
        add_monoid_algebra_neg_coeff(q, m)
        -(p.coeff(m) + q.coeff(m)) = -p.coeff(m) + -q.coeff(m)
        (p + q).neg.coeff(m) = (p.neg + q.neg).coeff(m)
    }
    add_monoid_algebra_ext_pointwise((p + q).neg, p.neg + q.neg)
}

/// Subtracting an additive monoid algebra element from itself gives zero.
theorem add_monoid_algebra_sub_self[R: Ring, M](p: AddMonoidAlgebra[R, M]) {
    p.sub(p) = AddMonoidAlgebra[R, M].zero
} by {
    add_monoid_algebra_add_neg_right(p)
}

/// Subtracting zero changes no additive monoid algebra element.
theorem add_monoid_algebra_sub_zero[R: Ring, M](p: AddMonoidAlgebra[R, M]) {
    p.sub(AddMonoidAlgebra[R, M].zero) = p
} by {
    add_monoid_algebra_neg_zero[R, M]
    add_monoid_algebra_add_zero_right(p)
}

/// Zero minus an additive monoid algebra element is its additive inverse.
theorem add_monoid_algebra_zero_sub[R: Ring, M](p: AddMonoidAlgebra[R, M]) {
    AddMonoidAlgebra[R, M].zero.sub(p) = p.neg
} by {
    add_monoid_algebra_add_zero_left(p.neg)
}

/// Addition by a fixed element on the left is cancellative.
theorem add_monoid_algebra_add_left_cancel[R: Ring, M](p: AddMonoidAlgebra[R, M], q: AddMonoidAlgebra[R, M], r: AddMonoidAlgebra[R, M]) {
    p + q = p + r implies q = r
} by {
    p + q = p + r
    forall(m: M) {
        add_monoid_algebra_eq_coeff_at(p + q, p + r, m)
        add_monoid_algebra_add_coeff(p, q, m)
        add_monoid_algebra_add_coeff(p, r, m)
        p.coeff(m) + q.coeff(m) = p.coeff(m) + r.coeff(m)
        q.coeff(m) = r.coeff(m)
    }
    add_monoid_algebra_ext_pointwise(q, r)
}

/// Addition by a fixed element on the right is cancellative.
theorem add_monoid_algebra_add_right_cancel[R: Ring, M](p: AddMonoidAlgebra[R, M], q: AddMonoidAlgebra[R, M], r: AddMonoidAlgebra[R, M]) {
    q + p = r + p implies q = r
} by {
    q + p = r + p
    forall(m: M) {
        add_monoid_algebra_eq_coeff_at(q + p, r + p, m)
        add_monoid_algebra_add_coeff(q, p, m)
        add_monoid_algebra_add_coeff(r, p, m)
        q.coeff(m) + p.coeff(m) = r.coeff(m) + p.coeff(m)
        q.coeff(m) = r.coeff(m)
    }
    add_monoid_algebra_ext_pointwise(q, r)
}

/// An additive monoid algebra element is zero when all its coefficients are zero.
theorem add_monoid_algebra_eq_zero_of_coeff_zero[R: Semiring, M](p: AddMonoidAlgebra[R, M]) {
    (forall(m: M) { p.coeff(m) = R.0 }) implies p = AddMonoidAlgebra[R, M].zero
} by {
    forall(m: M) {
        add_monoid_algebra_zero_coeff[R, M](m)
        p.coeff(m) = AddMonoidAlgebra[R, M].zero.coeff(m)
    }
    add_monoid_algebra_ext_pointwise(p, AddMonoidAlgebra[R, M].zero)
}

/// A zero additive monoid algebra element has zero coefficients.
theorem add_monoid_algebra_coeff_zero_of_eq_zero[R: Semiring, M](p: AddMonoidAlgebra[R, M], m: M) {
    p = AddMonoidAlgebra[R, M].zero implies p.coeff(m) = R.0
} by {
    add_monoid_algebra_eq_coeff_at(p, AddMonoidAlgebra[R, M].zero, m)
    add_monoid_algebra_zero_coeff[R, M](m)
}

/// If a sum is zero, then the sum of its coefficients is zero at every index.
theorem add_monoid_algebra_add_eq_zero_coeff[R: Semiring, M](p: AddMonoidAlgebra[R, M], q: AddMonoidAlgebra[R, M], m: M) {
    p + q = AddMonoidAlgebra[R, M].zero implies p.coeff(m) + q.coeff(m) = R.0
} by {
    add_monoid_algebra_eq_coeff_at(p + q, AddMonoidAlgebra[R, M].zero, m)
    add_monoid_algebra_add_coeff(p, q, m)
    add_monoid_algebra_zero_coeff[R, M](m)
}

/// Equal additive monoid algebra elements have zero difference.
theorem add_monoid_algebra_sub_eq_zero_of_eq[R: Ring, M](p: AddMonoidAlgebra[R, M], q: AddMonoidAlgebra[R, M]) {
    p = q implies p.sub(q) = AddMonoidAlgebra[R, M].zero
} by {
    add_monoid_algebra_sub_self(p)
}

/// An additive monoid algebra element is determined by a zero difference.
theorem add_monoid_algebra_eq_of_sub_eq_zero[R: Ring, M](p: AddMonoidAlgebra[R, M], q: AddMonoidAlgebra[R, M]) {
    p.sub(q) = AddMonoidAlgebra[R, M].zero implies p = q
} by {
    forall(m: M) {
        add_monoid_algebra_eq_coeff_at(p.sub(q), AddMonoidAlgebra[R, M].zero, m)
        add_monoid_algebra_sub_coeff(p, q, m)
        add_monoid_algebra_zero_coeff[R, M](m)
        p.coeff(m) + (-q.coeff(m) + q.coeff(m)) = (p.coeff(m) + -q.coeff(m)) + q.coeff(m)
        -q.coeff(m) + q.coeff(m) = R.0
        p.coeff(m) = q.coeff(m)
    }
    add_monoid_algebra_ext_pointwise(p, q)
}

/// Adding back a subtracted element gives the original element.
theorem add_monoid_algebra_sub_add_cancel[R: Ring, M](p: AddMonoidAlgebra[R, M], q: AddMonoidAlgebra[R, M]) {
    p.sub(q) + q = p
} by {
    forall(m: M) {
        add_monoid_algebra_add_coeff(p.sub(q), q, m)
        add_monoid_algebra_sub_coeff(p, q, m)
        (p.coeff(m) + -q.coeff(m)) + q.coeff(m) = p.coeff(m)
        (p.sub(q) + q).coeff(m) = p.coeff(m)
    }
    add_monoid_algebra_ext_pointwise(p.sub(q) + q, p)
}

/// Subtracting the right addend from a sum gives the left addend.
theorem add_monoid_algebra_add_sub_cancel[R: Ring, M](p: AddMonoidAlgebra[R, M], q: AddMonoidAlgebra[R, M]) {
    (p + q).sub(q) = p
} by {
    forall(m: M) {
        add_monoid_algebra_sub_coeff(p + q, q, m)
        add_monoid_algebra_add_coeff(p, q, m)
        (p.coeff(m) + q.coeff(m)) + -q.coeff(m) = p.coeff(m)
        (p + q).sub(q).coeff(m) = p.coeff(m)
    }
    add_monoid_algebra_ext_pointwise((p + q).sub(q), p)
}

/// Scalar multiplication commutes with additive inverse.
theorem add_monoid_algebra_smul_neg[R: Ring, M](r: R, p: AddMonoidAlgebra[R, M]) {
    p.neg.smul(r) = p.smul(r).neg
} by {
    forall(m: M) {
        add_monoid_algebra_smul_coeff(r, p.neg, m)
        add_monoid_algebra_neg_coeff(p, m)
        add_monoid_algebra_neg_coeff(p.smul(r), m)
        add_monoid_algebra_smul_coeff(r, p, m)
        r * -p.coeff(m) = -(r * p.coeff(m))
        p.neg.smul(r).coeff(m) = p.smul(r).neg.coeff(m)
    }
    add_monoid_algebra_ext_pointwise(p.neg.smul(r), p.smul(r).neg)
}

/// Scalar multiplication distributes over subtraction.
theorem add_monoid_algebra_smul_sub[R: Ring, M](r: R, p: AddMonoidAlgebra[R, M], q: AddMonoidAlgebra[R, M]) {
    p.sub(q).smul(r) = p.smul(r).sub(q.smul(r))
} by {
    forall(m: M) {
        add_monoid_algebra_smul_coeff(r, p.sub(q), m)
        add_monoid_algebra_sub_coeff(p, q, m)
        add_monoid_algebra_sub_coeff(p.smul(r), q.smul(r), m)
        add_monoid_algebra_smul_coeff(r, p, m)
        add_monoid_algebra_smul_coeff(r, q, m)
        r * (p.coeff(m) + -q.coeff(m)) = r * p.coeff(m) + -(r * q.coeff(m))
        p.sub(q).smul(r).coeff(m) = p.smul(r).sub(q.smul(r)).coeff(m)
    }
    add_monoid_algebra_ext_pointwise(p.sub(q).smul(r), p.smul(r).sub(q.smul(r)))
}

/// Scalar multiplication by the negated scalar is the additive inverse of scalar multiplication.
theorem add_monoid_algebra_neg_smul[R: Ring, M](r: R, p: AddMonoidAlgebra[R, M]) {
    p.smul(-r) = p.smul(r).neg
} by {
    forall(m: M) {
        add_monoid_algebra_smul_coeff(-r, p, m)
        add_monoid_algebra_neg_coeff(p.smul(r), m)
        add_monoid_algebra_smul_coeff(r, p, m)
        p.smul(-r).coeff(m) = p.smul(r).neg.coeff(m)
    }
    add_monoid_algebra_ext_pointwise(p.smul(-r), p.smul(r).neg)
}

/// Scalar multiplication of an additive inverse agrees with scalar multiplication by the negated scalar.
theorem add_monoid_algebra_smul_neg_eq_neg_smul[R: Ring, M](r: R, p: AddMonoidAlgebra[R, M]) {
    p.neg.smul(r) = p.smul(-r)
} by {
    add_monoid_algebra_smul_neg(r, p)
    add_monoid_algebra_neg_smul(r, p)
}

/// Scalar multiplication distributes over subtraction of scalars.
theorem add_monoid_algebra_sub_smul[R: Ring, M](r: R, s: R, p: AddMonoidAlgebra[R, M]) {
    p.smul(r + -s) = p.smul(r).sub(p.smul(s))
} by {
    forall(m: M) {
        add_monoid_algebra_smul_coeff(r + -s, p, m)
        add_monoid_algebra_sub_coeff(p.smul(r), p.smul(s), m)
        add_monoid_algebra_smul_coeff(r, p, m)
        add_monoid_algebra_smul_coeff(s, p, m)
        (r + -s) * p.coeff(m) = r * p.coeff(m) + -(s * p.coeff(m))
        p.smul(r + -s).coeff(m) = p.smul(r).sub(p.smul(s)).coeff(m)
    }
    add_monoid_algebra_ext_pointwise(p.smul(r + -s), p.smul(r).sub(p.smul(s)))
}

/// The additive inverse of a scalar multiple is the scalar multiple by the negated scalar.
theorem add_monoid_algebra_neg_smul_eq_smul_neg[R: Ring, M](r: R, p: AddMonoidAlgebra[R, M]) {
    p.smul(r).neg = p.smul(-r)
} by {
    add_monoid_algebra_neg_smul(r, p)
}

/// A single basis term with zero coefficient is zero at every index.
theorem add_monoid_algebra_single_zero_coeff[R: Semiring, M](m: M, n: M) {
    AddMonoidAlgebra[R, M].single(m, R.0).coeff(n) = R.0
} by {
    AddMonoidAlgebra[R, M].new(add_monoid_algebra_single_coeff[R, M](m, R.0)) =
        Option.some(AddMonoidAlgebra[R, M].single(m, R.0))
    AddMonoidAlgebra[R, M].single(m, R.0).coeff = add_monoid_algebra_single_coeff[R, M](m, R.0)
    add_monoid_algebra_single_coeff[R, M](m, R.0, n) = R.0
}

/// A single basis term with zero coefficient is the zero additive monoid algebra element.
theorem add_monoid_algebra_single_zero[R: Semiring, M](m: M) {
    AddMonoidAlgebra[R, M].single(m, R.0) = AddMonoidAlgebra[R, M].zero
} by {
    forall(n: M) {
        add_monoid_algebra_single_zero_coeff[R, M](m, n)
        add_monoid_algebra_zero_coeff[R, M](n)
        AddMonoidAlgebra[R, M].single(m, R.0).coeff(n) = AddMonoidAlgebra[R, M].zero.coeff(n)
    }
    add_monoid_algebra_ext_pointwise(AddMonoidAlgebra[R, M].single(m, R.0), AddMonoidAlgebra[R, M].zero)
}

/// A single basis term is zero exactly when its chosen coefficient is zero.
theorem add_monoid_algebra_single_eq_zero_iff[R: Semiring, M](m: M, r: R) {
    AddMonoidAlgebra[R, M].single(m, r) = AddMonoidAlgebra[R, M].zero = (r = R.0)
} by {
    if AddMonoidAlgebra[R, M].single(m, r) = AddMonoidAlgebra[R, M].zero {
        add_monoid_algebra_eq_coeff_at(AddMonoidAlgebra[R, M].single(m, r), AddMonoidAlgebra[R, M].zero, m)
        add_monoid_algebra_single_coeff_self[R, M](m, r)
        add_monoid_algebra_zero_coeff[R, M](m)
        r = R.0
    }
    if r = R.0 {
        add_monoid_algebra_single_zero[R, M](m)
        AddMonoidAlgebra[R, M].single(m, r) = AddMonoidAlgebra[R, M].zero
    }
}

/// Two single basis terms at the same basis index are equal exactly when their coefficients are equal.
theorem add_monoid_algebra_single_eq_single_same_iff[R: Semiring, M](m: M, r: R, s: R) {
    AddMonoidAlgebra[R, M].single(m, r) = AddMonoidAlgebra[R, M].single(m, s) = (r = s)
} by {
    if AddMonoidAlgebra[R, M].single(m, r) = AddMonoidAlgebra[R, M].single(m, s) {
        add_monoid_algebra_eq_coeff_at(AddMonoidAlgebra[R, M].single(m, r), AddMonoidAlgebra[R, M].single(m, s), m)
        add_monoid_algebra_single_coeff_self[R, M](m, r)
        add_monoid_algebra_single_coeff_self[R, M](m, s)
        r = s
    }
    if r = s {
        AddMonoidAlgebra[R, M].single(m, r) = AddMonoidAlgebra[R, M].single(m, s)
    }
}

/// Adding two single basis terms at their shared basis index adds the coefficients.
theorem add_monoid_algebra_single_add_same_coeff_self[R: Semiring, M](m: M, r: R, s: R) {
    (AddMonoidAlgebra[R, M].single(m, r) + AddMonoidAlgebra[R, M].single(m, s)).coeff(m) = r + s
} by {
    add_monoid_algebra_add_coeff(AddMonoidAlgebra[R, M].single(m, r), AddMonoidAlgebra[R, M].single(m, s), m)
    add_monoid_algebra_single_coeff_self[R, M](m, r)
    add_monoid_algebra_single_coeff_self[R, M](m, s)
}

/// Adding two single basis terms is zero away from their shared basis index.
theorem add_monoid_algebra_single_add_same_coeff_of_ne[R: Semiring, M](m: M, r: R, s: R, n: M) {
    n != m implies
    (AddMonoidAlgebra[R, M].single(m, r) + AddMonoidAlgebra[R, M].single(m, s)).coeff(n) = R.0
} by {
    add_monoid_algebra_add_coeff(AddMonoidAlgebra[R, M].single(m, r), AddMonoidAlgebra[R, M].single(m, s), n)
    add_monoid_algebra_single_coeff_of_ne[R, M](m, r, n)
    add_monoid_algebra_single_coeff_of_ne[R, M](m, s, n)
    R.0 + R.0 = R.0
}

/// Adding two single basis terms at the same basis index gives a single term.
theorem add_monoid_algebra_single_add_same[R: Semiring, M](m: M, r: R, s: R) {
    AddMonoidAlgebra[R, M].single(m, r) + AddMonoidAlgebra[R, M].single(m, s) =
        AddMonoidAlgebra[R, M].single(m, r + s)
} by {
    forall(n: M) {
        if n = m {
            add_monoid_algebra_single_add_same_coeff_self[R, M](m, r, s)
            add_monoid_algebra_single_coeff_self[R, M](m, r + s)
            (AddMonoidAlgebra[R, M].single(m, r) + AddMonoidAlgebra[R, M].single(m, s)).coeff(n) =
                AddMonoidAlgebra[R, M].single(m, r + s).coeff(n)
        } else {
            add_monoid_algebra_single_add_same_coeff_of_ne[R, M](m, r, s, n)
            add_monoid_algebra_single_coeff_of_ne[R, M](m, r + s, n)
            (AddMonoidAlgebra[R, M].single(m, r) + AddMonoidAlgebra[R, M].single(m, s)).coeff(n) =
                AddMonoidAlgebra[R, M].single(m, r + s).coeff(n)
        }
    }
    add_monoid_algebra_ext_pointwise(
        AddMonoidAlgebra[R, M].single(m, r) + AddMonoidAlgebra[R, M].single(m, s),
        AddMonoidAlgebra[R, M].single(m, r + s)
    )
}

/// Scalar multiplication of a single basis term scales its chosen coefficient.
theorem add_monoid_algebra_single_smul_coeff_self[R: Semiring, M](m: M, r: R, s: R) {
    AddMonoidAlgebra[R, M].single(m, r).smul(s).coeff(m) = s * r
} by {
    add_monoid_algebra_smul_coeff(s, AddMonoidAlgebra[R, M].single(m, r), m)
    add_monoid_algebra_single_coeff_self[R, M](m, r)
}

/// Scalar multiplication of a single basis term is zero away from its basis index.
theorem add_monoid_algebra_single_smul_coeff_of_ne[R: Semiring, M](m: M, r: R, s: R, n: M) {
    n != m implies AddMonoidAlgebra[R, M].single(m, r).smul(s).coeff(n) = R.0
} by {
    add_monoid_algebra_smul_coeff(s, AddMonoidAlgebra[R, M].single(m, r), n)
    add_monoid_algebra_single_coeff_of_ne[R, M](m, r, n)
    s * R.0 = R.0
}

/// Scalar multiplication of a single basis term is again a single basis term.
theorem add_monoid_algebra_single_smul[R: Semiring, M](m: M, r: R, s: R) {
    AddMonoidAlgebra[R, M].single(m, r).smul(s) = AddMonoidAlgebra[R, M].single(m, s * r)
} by {
    forall(n: M) {
        if n = m {
            add_monoid_algebra_single_smul_coeff_self[R, M](m, r, s)
            add_monoid_algebra_single_coeff_self[R, M](m, s * r)
            AddMonoidAlgebra[R, M].single(m, r).smul(s).coeff(n) =
                AddMonoidAlgebra[R, M].single(m, s * r).coeff(n)
        } else {
            add_monoid_algebra_single_smul_coeff_of_ne[R, M](m, r, s, n)
            add_monoid_algebra_single_coeff_of_ne[R, M](m, s * r, n)
            AddMonoidAlgebra[R, M].single(m, r).smul(s).coeff(n) =
                AddMonoidAlgebra[R, M].single(m, s * r).coeff(n)
        }
    }
    add_monoid_algebra_ext_pointwise(AddMonoidAlgebra[R, M].single(m, r).smul(s), AddMonoidAlgebra[R, M].single(m, s * r))
}

/// Negating a single basis term negates its chosen coefficient.
theorem add_monoid_algebra_single_neg_coeff_self[R: Ring, M](m: M, r: R) {
    AddMonoidAlgebra[R, M].single(m, r).neg.coeff(m) = -r
} by {
    add_monoid_algebra_neg_coeff(AddMonoidAlgebra[R, M].single(m, r), m)
    add_monoid_algebra_single_coeff_self[R, M](m, r)
}

/// Negating a single basis term is zero away from its basis index.
theorem add_monoid_algebra_single_neg_coeff_of_ne[R: Ring, M](m: M, r: R, n: M) {
    n != m implies AddMonoidAlgebra[R, M].single(m, r).neg.coeff(n) = R.0
} by {
    add_monoid_algebra_neg_coeff(AddMonoidAlgebra[R, M].single(m, r), n)
    add_monoid_algebra_single_coeff_of_ne[R, M](m, r, n)
    -R.0 = R.0
}

/// Negating a single basis term is again a single basis term.
theorem add_monoid_algebra_single_neg[R: Ring, M](m: M, r: R) {
    AddMonoidAlgebra[R, M].single(m, r).neg = AddMonoidAlgebra[R, M].single(m, -r)
} by {
    forall(n: M) {
        if n = m {
            add_monoid_algebra_single_neg_coeff_self[R, M](m, r)
            add_monoid_algebra_single_coeff_self[R, M](m, -r)
            AddMonoidAlgebra[R, M].single(m, r).neg.coeff(n) =
                AddMonoidAlgebra[R, M].single(m, -r).coeff(n)
        } else {
            add_monoid_algebra_single_neg_coeff_of_ne[R, M](m, r, n)
            add_monoid_algebra_single_coeff_of_ne[R, M](m, -r, n)
            AddMonoidAlgebra[R, M].single(m, r).neg.coeff(n) =
                AddMonoidAlgebra[R, M].single(m, -r).coeff(n)
        }
    }
    add_monoid_algebra_ext_pointwise(AddMonoidAlgebra[R, M].single(m, r).neg, AddMonoidAlgebra[R, M].single(m, -r))
}

/// Subtracting two single basis terms at their shared basis index subtracts the coefficients.
theorem add_monoid_algebra_single_sub_same_coeff_self[R: Ring, M](m: M, r: R, s: R) {
    AddMonoidAlgebra[R, M].single(m, r).sub(AddMonoidAlgebra[R, M].single(m, s)).coeff(m) = r + -s
} by {
    add_monoid_algebra_sub_coeff(AddMonoidAlgebra[R, M].single(m, r), AddMonoidAlgebra[R, M].single(m, s), m)
    add_monoid_algebra_single_coeff_self[R, M](m, r)
    add_monoid_algebra_single_coeff_self[R, M](m, s)
}

/// Subtracting two single basis terms is zero away from their shared basis index.
theorem add_monoid_algebra_single_sub_same_coeff_of_ne[R: Ring, M](m: M, r: R, s: R, n: M) {
    n != m implies AddMonoidAlgebra[R, M].single(m, r).sub(AddMonoidAlgebra[R, M].single(m, s)).coeff(n) = R.0
} by {
    add_monoid_algebra_sub_coeff(AddMonoidAlgebra[R, M].single(m, r), AddMonoidAlgebra[R, M].single(m, s), n)
    add_monoid_algebra_single_coeff_of_ne[R, M](m, r, n)
    add_monoid_algebra_single_coeff_of_ne[R, M](m, s, n)
    R.0 + -R.0 = R.0
}

/// Subtracting two single basis terms at the same basis index gives a single term.
theorem add_monoid_algebra_single_sub_same[R: Ring, M](m: M, r: R, s: R) {
    AddMonoidAlgebra[R, M].single(m, r).sub(AddMonoidAlgebra[R, M].single(m, s)) =
        AddMonoidAlgebra[R, M].single(m, r + -s)
} by {
    forall(n: M) {
        if n = m {
            add_monoid_algebra_single_sub_same_coeff_self[R, M](m, r, s)
            add_monoid_algebra_single_coeff_self[R, M](m, r + -s)
            AddMonoidAlgebra[R, M].single(m, r).sub(AddMonoidAlgebra[R, M].single(m, s)).coeff(n) =
                AddMonoidAlgebra[R, M].single(m, r + -s).coeff(n)
        } else {
            add_monoid_algebra_single_sub_same_coeff_of_ne[R, M](m, r, s, n)
            add_monoid_algebra_single_coeff_of_ne[R, M](m, r + -s, n)
            AddMonoidAlgebra[R, M].single(m, r).sub(AddMonoidAlgebra[R, M].single(m, s)).coeff(n) =
                AddMonoidAlgebra[R, M].single(m, r + -s).coeff(n)
        }
    }
    add_monoid_algebra_ext_pointwise(
        AddMonoidAlgebra[R, M].single(m, r).sub(AddMonoidAlgebra[R, M].single(m, s)),
        AddMonoidAlgebra[R, M].single(m, r + -s)
    )
}
