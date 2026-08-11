// Group theory deepening: basic group laws not yet stated elsewhere —
// uniqueness of inverses, cancellation against inverses, injectivity of the
// inverse operation, products equal to the identity, powers of an element,
// and the commutation laws that follow from commutativity of the group.

from algebra.group import Group, GroupHom, inverse_left, inverse_inverse, inverse_mul, left_cancel
from algebra.comm_group import CommGroup
from algebra.group.group_commute import commute, commute_eq
from algebra.subgroup import Subgroup, group_hom_kernel, subgroup_contains_identity
from algebra.group.group_hom_subgroup import subgroup_image, subgroup_image_identity_constraint
from nat import Nat, pow_one, pow_add, pow_zero, alt_induction
numerals Nat

/// The identity element of a group is its own inverse.
theorem group_inverse_one[G: Group] {
    G.1.inverse = G.1
} by {
    inverse_left(G.1)
    G.1.inverse * G.1 = G.1
    G.1.inverse * G.1 = G.1.inverse
    G.1.inverse = G.1
}

/// If a product is the identity on both sides, the factor is the inverse.
theorem group_inverse_unique[G: Group](a: G, b: G) {
    a * b = G.1 and b * a = G.1 implies b = a.inverse
} by {
    if a * b = G.1 and b * a = G.1 {
        inverse_left(a)
        a.inverse * (a * b) = a.inverse * G.1
        a.inverse * (a * b) = (a.inverse * a) * b
        a.inverse * a = G.1
        (a.inverse * a) * b = G.1 * b
        G.1 * b = b
        a.inverse * G.1 = a.inverse
        b = a.inverse
    }
}

/// A right inverse is the inverse: a * b = 1 implies b = a.inverse.
theorem group_right_inverse_eq_inverse[G: Group](a: G, b: G) {
    a * b = G.1 implies b = a.inverse
} by {
    if a * b = G.1 {
        inverse_left(a)
        a.inverse * (a * b) = a.inverse * G.1
        a.inverse * (a * b) = (a.inverse * a) * b
        a.inverse * a = G.1
        (a.inverse * a) * b = G.1 * b
        G.1 * b = b
        a.inverse * G.1 = a.inverse
        b = a.inverse
    }
}

/// A left inverse is the inverse: b * a = 1 implies b = a.inverse.
theorem group_left_inverse_eq_inverse[G: Group](a: G, b: G) {
    b * a = G.1 implies b = a.inverse
} by {
    if b * a = G.1 {
        (b * a) * a.inverse = G.1 * a.inverse
        (b * a) * a.inverse = b * (a * a.inverse)
        a * a.inverse = G.1
        b * (a * a.inverse) = b * G.1
        b * G.1 = b
        G.1 * a.inverse = a.inverse
        b = a.inverse
    }
}

/// The inverse of an inverse-times-element is the inverse times the inverse.
theorem group_inverse_mul_inv_left[G: Group](a: G, b: G) {
    (a.inverse * b).inverse = b.inverse * a
} by {
    inverse_mul(a.inverse, b)
    (a.inverse * b).inverse = b.inverse * a.inverse.inverse
    inverse_inverse(a)
    a.inverse.inverse = a
    b.inverse * a.inverse.inverse = b.inverse * a
}

/// The inverse of an element-times-inverse is the inverse times the inverse.
theorem group_inverse_mul_inv_right[G: Group](a: G, b: G) {
    (a * b.inverse).inverse = b * a.inverse
} by {
    inverse_mul(a, b.inverse)
    (a * b.inverse).inverse = b.inverse.inverse * a.inverse
    inverse_inverse(b)
    b.inverse.inverse = b
    b.inverse.inverse * a.inverse = b * a.inverse
}

/// Cancelling a factor against its inverse on the left.
theorem group_mul_inv_cancel_left[G: Group](a: G, b: G) {
    a * (a.inverse * b) = b
} by {
    a * (a.inverse * b) = (a * a.inverse) * b
    a * a.inverse = G.1
    (a * a.inverse) * b = G.1 * b
    G.1 * b = b
    a * (a.inverse * b) = b
}

/// Cancelling a factor against its inverse on the right.
theorem group_mul_inv_cancel_right[G: Group](a: G, b: G) {
    (b * a) * a.inverse = b
} by {
    (b * a) * a.inverse = b * (a * a.inverse)
    a * a.inverse = G.1
    b * (a * a.inverse) = b * G.1
    b * G.1 = b
    (b * a) * a.inverse = b
}

/// Cancelling an inverse against its factor on the left.
theorem group_inv_mul_cancel_left[G: Group](a: G, b: G) {
    a.inverse * (a * b) = b
} by {
    a.inverse * (a * b) = (a.inverse * a) * b
    inverse_left(a)
    a.inverse * a = G.1
    (a.inverse * a) * b = G.1 * b
    G.1 * b = b
    a.inverse * (a * b) = b
}

/// Cancelling an inverse against its factor on the right.
theorem group_inv_mul_cancel_right[G: Group](a: G, b: G) {
    (b * a.inverse) * a = b
} by {
    (b * a.inverse) * a = b * (a.inverse * a)
    inverse_left(a)
    a.inverse * a = G.1
    b * (a.inverse * a) = b * G.1
    b * G.1 = b
    (b * a.inverse) * a = b
}

/// A product equals the identity exactly when the right factor is the inverse of the left.
theorem group_mul_eq_one_iff_eq_inverse[G: Group](a: G, b: G) {
    (a * b = G.1) = (b = a.inverse)
} by {
    if a * b = G.1 {
        group_right_inverse_eq_inverse(a, b)
        b = a.inverse
    }
    if b = a.inverse {
        a * b = a * a.inverse
        a * a.inverse = G.1
        a * b = G.1
    }
    (a * b = G.1) = (b = a.inverse)
}

/// A product equals the identity exactly when the left factor is the inverse of the right.
theorem group_mul_eq_one_iff_eq_inverse_left[G: Group](a: G, b: G) {
    (a * b = G.1) = (a = b.inverse)
} by {
    if a * b = G.1 {
        group_left_inverse_eq_inverse(b, a)
        a = b.inverse
    }
    if a = b.inverse {
        a * b = b.inverse * b
        inverse_left(b)
        b.inverse * b = G.1
        a * b = G.1
    }
    (a * b = G.1) = (a = b.inverse)
}

/// The inverse operation is injective.
theorem group_inverse_injective[G: Group](a: G, b: G) {
    a.inverse = b.inverse implies a = b
} by {
    if a.inverse = b.inverse {
        inverse_inverse(a)
        a.inverse.inverse = a
        a.inverse.inverse = b.inverse.inverse
        inverse_inverse(b)
        b.inverse.inverse = b
        a = b
    }
}

/// Two elements are equal exactly when their inverses are equal.
theorem group_eq_iff_inv_eq[G: Group](a: G, b: G) {
    (a = b) = (a.inverse = b.inverse)
} by {
    if a = b {
        a.inverse = b.inverse
    }
    if a.inverse = b.inverse {
        group_inverse_injective(a, b)
        a = b
    }
    (a = b) = (a.inverse = b.inverse)
}

/// An element equals its inverse exactly when it squares to the identity.
theorem group_sq_eq_one_iff_inv_eq_self[G: Group](a: G) {
    (a * a = G.1) = (a.inverse = a)
} by {
    if a * a = G.1 {
        group_right_inverse_eq_inverse(a, a)
        a = a.inverse
    }
    if a.inverse = a {
        a * a = a * a.inverse
        a * a.inverse = G.1
        a * a = G.1
    }
    (a * a = G.1) = (a.inverse = a)
}

/// The square of an element is the product of the element with itself.
theorem group_pow_two[G: Group](a: G) {
    a.pow(2) = a * a
} by {
    pow_add(a, Nat.1, Nat.1)
    a.pow(Nat.1) * a.pow(Nat.1) = a.pow(2)
    pow_one(a)
    a.pow(Nat.1) = a
    a * a = a.pow(2)
}

/// The cube of an element is the product of the element with itself twice.
theorem group_pow_three[G: Group](a: G) {
    a.pow(3) = a * a * a
} by {
    pow_add(a, Nat.2, Nat.1)
    a.pow(Nat.2) * a.pow(Nat.1) = a.pow(3)
    group_pow_two(a)
    a.pow(2) = a * a
    pow_one(a)
    a.pow(Nat.1) = a
    a * a * a = a.pow(3)
}

/// Powers of the same element commute: a^m * a^n = a^n * a^m.
theorem group_pow_comm[G: Group](a: G, m: Nat, n: Nat) {
    a.pow(m) * a.pow(n) = a.pow(n) * a.pow(m)
} by {
    pow_add(a, m, n)
    a.pow(m) * a.pow(n) = a.pow(m + n)
    pow_add(a, n, m)
    a.pow(n) * a.pow(m) = a.pow(n + m)
    m + n = n + m
    a.pow(m + n) = a.pow(n + m)
    a.pow(m) * a.pow(n) = a.pow(n) * a.pow(m)
}

/// In a commutative group every pair of elements commutes.
theorem comm_group_commute[G: CommGroup](a: G, b: G) {
    commute[G](a, b)
} by {
    commute_eq[G](a, b)
    commute[G](a, b) = (a * b = b * a)
    a * b = b * a
    commute[G](a, b)
}

/// In a commutative group the inverse of a product is the product of the inverses.
theorem comm_group_inverse_mul[G: CommGroup](a: G, b: G) {
    (a * b).inverse = a.inverse * b.inverse
} by {
    inverse_mul(a, b)
    (a * b).inverse = b.inverse * a.inverse
    b.inverse * a.inverse = a.inverse * b.inverse
}

/// An element equals its inverse in a commutative group exactly when it squares to the identity.
theorem comm_group_inv_eq_self_iff_sq_one[G: CommGroup](a: G) {
    (a.inverse = a) = (a * a = G.1)
} by {
    group_sq_eq_one_iff_inv_eq_self(a)
    (a * a = G.1) = (a.inverse = a)
}

/// The kernel of a group homomorphism contains the identity.
theorem group_hom_kernel_contains_identity[G: Group, H: Group](f: GroupHom[G, H]) {
    group_hom_kernel(f).contains(G.1)
} by {
    subgroup_contains_identity(group_hom_kernel(f))
    group_hom_kernel(f).contains(G.1)
}

/// The image of a subgroup under a group homomorphism contains the identity.
theorem group_hom_image_contains_identity[G: Group, H: Group](f: GroupHom[G, H], s: Subgroup[G]) {
    subgroup_image(f, s).contains(H.1)
} by {
    subgroup_image_identity_constraint(f, s)
    subgroup_image(f, s).contains(H.1)
}

/// An element commutes with every power of an element it commutes with.
theorem group_commute_pow_right[G: Group](a: G, b: G, n: Nat) {
    a * b = b * a implies a * b.pow(n) = b.pow(n) * a
} by {
    define p(k: Nat) -> Bool {
        a * b = b * a implies a * b.pow(k) = b.pow(k) * a
    }
    pow_zero(b)
    b.pow(Nat.0) = G.1
    a * b.pow(Nat.0) = a * G.1
    a * G.1 = a
    b.pow(Nat.0) * a = G.1 * a
    G.1 * a = a
    a * b.pow(Nat.0) = b.pow(Nat.0) * a
    if a * b = b * a {
        a * b.pow(Nat.0) = b.pow(Nat.0) * a
    }
    p(Nat.0)
    forall(k: Nat) {
        if p(k) {
            if a * b = b * a {
                pow_add(b, k, Nat.1)
                b.pow(k) * b.pow(Nat.1) = b.pow(k + Nat.1)
                pow_one(b)
                b.pow(Nat.1) = b
                k + Nat.1 = k.suc
                b.pow(k + Nat.1) = b.pow(k.suc)
                b.pow(k) * b = b.pow(k.suc)
                a * b.pow(k.suc) = a * (b.pow(k) * b)
                a * (b.pow(k) * b) = (a * b.pow(k)) * b
                a * b.pow(k) = b.pow(k) * a
                (a * b.pow(k)) * b = (b.pow(k) * a) * b
                (b.pow(k) * a) * b = b.pow(k) * (a * b)
                a * b = b * a
                b.pow(k) * (a * b) = b.pow(k) * (b * a)
                b.pow(k) * (b * a) = (b.pow(k) * b) * a
                b.pow(k) * b = b.pow(k.suc)
                (b.pow(k) * b) * a = b.pow(k.suc) * a
                a * b.pow(k.suc) = b.pow(k.suc) * a
            }
            p(k.suc)
        }
    }
    p(Nat.0) and forall(k: Nat) { p(k) implies p(k.suc) }
    alt_induction(p)
    forall(k: Nat) { p(k) }
}
