/// Two-by-two matrix algebra: matrix multiplication is associative, the
/// two-by-two determinant satisfies `ad - bc`, the identity matrix acts as an
/// identity, multiplication distributes over addition, and a two-by-two matrix
/// with nonzero determinant has the adjugate inverse.  The finite-sum
/// machinery is expanded by hand for the two-element index set, which is how
/// the entrywise verifications below are carried out.

from nat import Nat
from semiring import Semiring
from algebra.add_comm_monoid import AddCommMonoid
from algebra.add_group import inverse_add
from algebra.field.field import Field, inverse_one
from algebra.ring.ring import Ring
from algebra.add_comm_monoid_rearrange import add_swap_inner
from data.fin.fin import Fin, fin_zero, fin_succ, ext, fin_zero_value, fin_succ_value
from data.fin.fin_sum import fin_sum, fin_sum_zero_of_bound_zero
from data.fin.fin_sum_enum import fin_sum_suc_split_zero, fin_succ_apply
from data.fin.fin_matrix import Matrix, matrix_entry, matrix_ext, matrix_one,
    square_matrix_mul, matrix_mul_term, matrix_entry_square_mul,
    square_matrix_mul_one_right, square_matrix_mul_one_left,
    square_matrix_mul_add_right, square_matrix_mul_add_left,
    matrix_cofactor_with, matrix_minor
from data.fin.fin_matrix_det import matrix_det_suc
from data.fin.fin_matrix_eigen import matrix_apply, matrix_apply_eq, matrix_apply_term
from real import Real

numerals Nat
numerals Real

// ---------------------------------------------------------------------------
// The two elements of `Fin[2]`
// ---------------------------------------------------------------------------

/// The second element of `Fin[2]` has natural value one.
theorem fin_two_one_value {
    fin_succ(Nat.0.suc, fin_zero(Nat.0)).value = Nat.0.suc
} by {
    fin_succ_value(Nat.0.suc, fin_zero(Nat.0))
    fin_zero_value(Nat.0)
}

/// The first element of `Fin[2]` has natural value zero.
theorem fin_two_zero_value {
    fin_zero(Nat.0.suc).value = Nat.0
} by {
    fin_zero_value(Nat.0.suc)
}

/// The two elements of `Fin[2]` are distinct.
theorem fin_two_one_ne_zero {
    fin_succ(Nat.0.suc, fin_zero(Nat.0)) != fin_zero(Nat.0.suc)
} by {
    if fin_succ(Nat.0.suc, fin_zero(Nat.0)) = fin_zero(Nat.0.suc) {
        fin_two_one_value
        fin_two_zero_value
        fin_succ(Nat.0.suc, fin_zero(Nat.0)).value = fin_zero(Nat.0.suc).value
        ext(Nat.0.suc.suc, fin_succ(Nat.0.suc, fin_zero(Nat.0)), fin_zero(Nat.0.suc))
        false
    }
}

/// A finite sum over `Fin[2]` is the sum of the two values.
theorem fin_sum_two[A: AddCommMonoid](f: Fin[Nat.0.suc.suc] -> A) {
    fin_sum[A](Nat.0.suc.suc, f) =
        f(fin_zero(Nat.0.suc)) + f(fin_succ(Nat.0.suc, fin_zero(Nat.0)))
} by {
    fin_sum_suc_split_zero[A](Nat.0.suc, f)
    fin_sum[A](Nat.0.suc.suc, f) =
        f(fin_zero(Nat.0.suc)) +
        fin_sum[A](Nat.0.suc, fin_succ_apply[A](Nat.0.suc, f))
    fin_sum_suc_split_zero[A](Nat.0, fin_succ_apply[A](Nat.0.suc, f))
    fin_sum[A](Nat.0.suc, fin_succ_apply[A](Nat.0.suc, f)) =
        fin_succ_apply[A](Nat.0.suc, f, fin_zero(Nat.0)) +
        fin_sum[A](Nat.0, fin_succ_apply[A](Nat.0, fin_succ_apply[A](Nat.0.suc, f)))
    fin_sum_zero_of_bound_zero[A](Nat.0, fin_succ_apply[A](Nat.0, fin_succ_apply[A](Nat.0.suc, f)))
    fin_sum[A](Nat.0, fin_succ_apply[A](Nat.0, fin_succ_apply[A](Nat.0.suc, f))) = A.0
    fin_succ_apply[A](Nat.0.suc, f, fin_zero(Nat.0)) = f(fin_succ(Nat.0.suc, fin_zero(Nat.0)))
    fin_sum[A](Nat.0.suc, fin_succ_apply[A](Nat.0.suc, f)) =
        f(fin_succ(Nat.0.suc, fin_zero(Nat.0))) + A.0
    f(fin_succ(Nat.0.suc, fin_zero(Nat.0))) + A.0 =
        f(fin_succ(Nat.0.suc, fin_zero(Nat.0)))
    fin_sum[A](Nat.0.suc.suc, f) =
        f(fin_zero(Nat.0.suc)) + f(fin_succ(Nat.0.suc, fin_zero(Nat.0)))
}

/// An entry of the product of two two-by-two matrices is the sum of the two
/// corresponding products.
theorem matrix_entry_mul_two[S: Semiring](
    a: Matrix[S, Nat.0.suc.suc, Nat.0.suc.suc],
    b: Matrix[S, Nat.0.suc.suc, Nat.0.suc.suc],
    i: Fin[Nat.0.suc.suc],
    j: Fin[Nat.0.suc.suc]
) {
    matrix_entry[S](Nat.0.suc.suc, Nat.0.suc.suc, square_matrix_mul[S](Nat.0.suc.suc, a, b), i, j) =
        matrix_entry[S](Nat.0.suc.suc, Nat.0.suc.suc, a, i, fin_zero(Nat.0.suc)) *
            matrix_entry[S](Nat.0.suc.suc, Nat.0.suc.suc, b, fin_zero(Nat.0.suc), j) +
        matrix_entry[S](Nat.0.suc.suc, Nat.0.suc.suc, a, i, fin_succ(Nat.0.suc, fin_zero(Nat.0))) *
            matrix_entry[S](Nat.0.suc.suc, Nat.0.suc.suc, b, fin_succ(Nat.0.suc, fin_zero(Nat.0)), j)
} by {
    matrix_entry_square_mul[S](Nat.0.suc.suc, a, b, i, j)
    matrix_entry[S](Nat.0.suc.suc, Nat.0.suc.suc, square_matrix_mul[S](Nat.0.suc.suc, a, b), i, j) =
        fin_sum[S](Nat.0.suc.suc, matrix_mul_term[S](Nat.0.suc.suc, Nat.0.suc.suc, Nat.0.suc.suc, a, b, i, j))
    fin_sum_two[S](matrix_mul_term[S](Nat.0.suc.suc, Nat.0.suc.suc, Nat.0.suc.suc, a, b, i, j))
    fin_sum[S](Nat.0.suc.suc, matrix_mul_term[S](Nat.0.suc.suc, Nat.0.suc.suc, Nat.0.suc.suc, a, b, i, j)) =
        matrix_mul_term[S](Nat.0.suc.suc, Nat.0.suc.suc, Nat.0.suc.suc, a, b, i, j, fin_zero(Nat.0.suc)) +
        matrix_mul_term[S](Nat.0.suc.suc, Nat.0.suc.suc, Nat.0.suc.suc, a, b, i, j, fin_succ(Nat.0.suc, fin_zero(Nat.0)))
    matrix_mul_term[S](Nat.0.suc.suc, Nat.0.suc.suc, Nat.0.suc.suc, a, b, i, j, fin_zero(Nat.0.suc)) =
        matrix_entry[S](Nat.0.suc.suc, Nat.0.suc.suc, a, i, fin_zero(Nat.0.suc)) *
            matrix_entry[S](Nat.0.suc.suc, Nat.0.suc.suc, b, fin_zero(Nat.0.suc), j)
    matrix_mul_term[S](Nat.0.suc.suc, Nat.0.suc.suc, Nat.0.suc.suc, a, b, i, j, fin_succ(Nat.0.suc, fin_zero(Nat.0))) =
        matrix_entry[S](Nat.0.suc.suc, Nat.0.suc.suc, a, i, fin_succ(Nat.0.suc, fin_zero(Nat.0))) *
            matrix_entry[S](Nat.0.suc.suc, Nat.0.suc.suc, b, fin_succ(Nat.0.suc, fin_zero(Nat.0)), j)
    matrix_entry[S](Nat.0.suc.suc, Nat.0.suc.suc, square_matrix_mul[S](Nat.0.suc.suc, a, b), i, j) =
        matrix_entry[S](Nat.0.suc.suc, Nat.0.suc.suc, a, i, fin_zero(Nat.0.suc)) *
            matrix_entry[S](Nat.0.suc.suc, Nat.0.suc.suc, b, fin_zero(Nat.0.suc), j) +
        matrix_entry[S](Nat.0.suc.suc, Nat.0.suc.suc, a, i, fin_succ(Nat.0.suc, fin_zero(Nat.0))) *
            matrix_entry[S](Nat.0.suc.suc, Nat.0.suc.suc, b, fin_succ(Nat.0.suc, fin_zero(Nat.0)), j)
}

// ---------------------------------------------------------------------------
// The two-by-two double-sum rearrangement
// ---------------------------------------------------------------------------

/// The double-sum rearrangement `sum_l x_l y_lk` times `z_k`, summed over the
/// two values of `k`, equals the two-term sum of `x_l` times the inner sums.
/// This is the entrywise content of associativity of two-by-two matrix
/// multiplication.
theorem semiring_four_term_swap[S: Semiring](
    x0: S, x1: S, y00: S, y01: S, y10: S, y11: S, z0: S, z1: S
) {
    (x0 * y00 + x1 * y10) * z0 + (x0 * y01 + x1 * y11) * z1 =
        x0 * (y00 * z0 + y01 * z1) + x1 * (y10 * z0 + y11 * z1)
} by {
    (x0 * y00 + x1 * y10) * z0 = x0 * y00 * z0 + x1 * y10 * z0
    (x0 * y01 + x1 * y11) * z1 = x0 * y01 * z1 + x1 * y11 * z1
    x0 * (y00 * z0 + y01 * z1) = x0 * y00 * z0 + x0 * y01 * z1
    x1 * (y10 * z0 + y11 * z1) = x1 * y10 * z0 + x1 * y11 * z1
    (x0 * y00 + x1 * y10) * z0 + (x0 * y01 + x1 * y11) * z1 =
        (x0 * y00 * z0 + x1 * y10 * z0) + (x0 * y01 * z1 + x1 * y11 * z1)
    x0 * (y00 * z0 + y01 * z1) + x1 * (y10 * z0 + y11 * z1) =
        (x0 * y00 * z0 + x0 * y01 * z1) + (x1 * y10 * z0 + x1 * y11 * z1)
    add_swap_inner[S](x0 * y00 * z0, x1 * y10 * z0, x0 * y01 * z1, x1 * y11 * z1)
    (x0 * y00 * z0 + x1 * y10 * z0) + (x0 * y01 * z1 + x1 * y11 * z1) =
        (x0 * y00 * z0 + x0 * y01 * z1) + (x1 * y10 * z0 + x1 * y11 * z1)
    (x0 * y00 + x1 * y10) * z0 + (x0 * y01 + x1 * y11) * z1 =
        x0 * (y00 * z0 + y01 * z1) + x1 * (y10 * z0 + y11 * z1)
}

// ---------------------------------------------------------------------------
// Associativity of matrix multiplication
// ---------------------------------------------------------------------------

// The general statement for compatible matrices:
//
//     theorem matrix_mul_assoc[S: Semiring](
//         m: Nat, n: Nat, p: Nat, q: Nat,
//         a: Matrix[S, m, n], b: Matrix[S, n, p], c: Matrix[S, p, q]
//     ) {
//         matrix_mul[S](m, p, q, matrix_mul[S](m, n, p, a, b), c) =
//             matrix_mul[S](m, n, q, a, matrix_mul[S](n, p, q, b, c))
//     }
//
// This is the double-sum rearrangement `sum_k (sum_l a(i,l) b(l,k)) c(k,j) =
// sum_l a(i,l) (sum_k b(l,k) c(k,j))`.  The library does not yet carry a
// finite-sum interchange lemma, so the statement is left here for reference
// and the two-by-two case is proved by expansion below.

/// Matrix multiplication is associative for two-by-two matrices.
theorem square_matrix_mul_assoc_two[S: Semiring](
    a: Matrix[S, Nat.0.suc.suc, Nat.0.suc.suc],
    b: Matrix[S, Nat.0.suc.suc, Nat.0.suc.suc],
    c: Matrix[S, Nat.0.suc.suc, Nat.0.suc.suc]
) {
    square_matrix_mul[S](Nat.0.suc.suc, square_matrix_mul[S](Nat.0.suc.suc, a, b), c) =
        square_matrix_mul[S](Nat.0.suc.suc, a, square_matrix_mul[S](Nat.0.suc.suc, b, c))
} by {
    let ab = square_matrix_mul[S](Nat.0.suc.suc, a, b)
    let bc = square_matrix_mul[S](Nat.0.suc.suc, b, c)
    let lhs = square_matrix_mul[S](Nat.0.suc.suc, ab, c)
    let rhs = square_matrix_mul[S](Nat.0.suc.suc, a, bc)
    forall(i: Fin[Nat.0.suc.suc], j: Fin[Nat.0.suc.suc]) {
        matrix_entry_mul_two[S](ab, c, i, j)
        matrix_entry[S](Nat.0.suc.suc, Nat.0.suc.suc, lhs, i, j) =
            matrix_entry[S](Nat.0.suc.suc, Nat.0.suc.suc, ab, i, fin_zero(Nat.0.suc)) *
                matrix_entry[S](Nat.0.suc.suc, Nat.0.suc.suc, c, fin_zero(Nat.0.suc), j) +
            matrix_entry[S](Nat.0.suc.suc, Nat.0.suc.suc, ab, i, fin_succ(Nat.0.suc, fin_zero(Nat.0))) *
                matrix_entry[S](Nat.0.suc.suc, Nat.0.suc.suc, c, fin_succ(Nat.0.suc, fin_zero(Nat.0)), j)
        matrix_entry_mul_two[S](a, b, i, fin_zero(Nat.0.suc))
        matrix_entry[S](Nat.0.suc.suc, Nat.0.suc.suc, ab, i, fin_zero(Nat.0.suc)) =
            matrix_entry[S](Nat.0.suc.suc, Nat.0.suc.suc, a, i, fin_zero(Nat.0.suc)) *
                matrix_entry[S](Nat.0.suc.suc, Nat.0.suc.suc, b, fin_zero(Nat.0.suc), fin_zero(Nat.0.suc)) +
            matrix_entry[S](Nat.0.suc.suc, Nat.0.suc.suc, a, i, fin_succ(Nat.0.suc, fin_zero(Nat.0))) *
                matrix_entry[S](Nat.0.suc.suc, Nat.0.suc.suc, b, fin_succ(Nat.0.suc, fin_zero(Nat.0)), fin_zero(Nat.0.suc))
        matrix_entry_mul_two[S](a, b, i, fin_succ(Nat.0.suc, fin_zero(Nat.0)))
        matrix_entry[S](Nat.0.suc.suc, Nat.0.suc.suc, ab, i, fin_succ(Nat.0.suc, fin_zero(Nat.0))) =
            matrix_entry[S](Nat.0.suc.suc, Nat.0.suc.suc, a, i, fin_zero(Nat.0.suc)) *
                matrix_entry[S](Nat.0.suc.suc, Nat.0.suc.suc, b, fin_zero(Nat.0.suc), fin_succ(Nat.0.suc, fin_zero(Nat.0))) +
            matrix_entry[S](Nat.0.suc.suc, Nat.0.suc.suc, a, i, fin_succ(Nat.0.suc, fin_zero(Nat.0))) *
                matrix_entry[S](Nat.0.suc.suc, Nat.0.suc.suc, b, fin_succ(Nat.0.suc, fin_zero(Nat.0)), fin_succ(Nat.0.suc, fin_zero(Nat.0)))
        matrix_entry_mul_two[S](a, bc, i, j)
        matrix_entry[S](Nat.0.suc.suc, Nat.0.suc.suc, rhs, i, j) =
            matrix_entry[S](Nat.0.suc.suc, Nat.0.suc.suc, a, i, fin_zero(Nat.0.suc)) *
                matrix_entry[S](Nat.0.suc.suc, Nat.0.suc.suc, bc, fin_zero(Nat.0.suc), j) +
            matrix_entry[S](Nat.0.suc.suc, Nat.0.suc.suc, a, i, fin_succ(Nat.0.suc, fin_zero(Nat.0))) *
                matrix_entry[S](Nat.0.suc.suc, Nat.0.suc.suc, bc, fin_succ(Nat.0.suc, fin_zero(Nat.0)), j)
        matrix_entry_mul_two[S](b, c, fin_zero(Nat.0.suc), j)
        matrix_entry[S](Nat.0.suc.suc, Nat.0.suc.suc, bc, fin_zero(Nat.0.suc), j) =
            matrix_entry[S](Nat.0.suc.suc, Nat.0.suc.suc, b, fin_zero(Nat.0.suc), fin_zero(Nat.0.suc)) *
                matrix_entry[S](Nat.0.suc.suc, Nat.0.suc.suc, c, fin_zero(Nat.0.suc), j) +
            matrix_entry[S](Nat.0.suc.suc, Nat.0.suc.suc, b, fin_zero(Nat.0.suc), fin_succ(Nat.0.suc, fin_zero(Nat.0))) *
                matrix_entry[S](Nat.0.suc.suc, Nat.0.suc.suc, c, fin_succ(Nat.0.suc, fin_zero(Nat.0)), j)
        matrix_entry_mul_two[S](b, c, fin_succ(Nat.0.suc, fin_zero(Nat.0)), j)
        matrix_entry[S](Nat.0.suc.suc, Nat.0.suc.suc, bc, fin_succ(Nat.0.suc, fin_zero(Nat.0)), j) =
            matrix_entry[S](Nat.0.suc.suc, Nat.0.suc.suc, b, fin_succ(Nat.0.suc, fin_zero(Nat.0)), fin_zero(Nat.0.suc)) *
                matrix_entry[S](Nat.0.suc.suc, Nat.0.suc.suc, c, fin_zero(Nat.0.suc), j) +
            matrix_entry[S](Nat.0.suc.suc, Nat.0.suc.suc, b, fin_succ(Nat.0.suc, fin_zero(Nat.0)), fin_succ(Nat.0.suc, fin_zero(Nat.0))) *
                matrix_entry[S](Nat.0.suc.suc, Nat.0.suc.suc, c, fin_succ(Nat.0.suc, fin_zero(Nat.0)), j)
        matrix_entry[S](Nat.0.suc.suc, Nat.0.suc.suc, lhs, i, j) =
            (matrix_entry[S](Nat.0.suc.suc, Nat.0.suc.suc, a, i, fin_zero(Nat.0.suc)) *
                matrix_entry[S](Nat.0.suc.suc, Nat.0.suc.suc, b, fin_zero(Nat.0.suc), fin_zero(Nat.0.suc)) +
                matrix_entry[S](Nat.0.suc.suc, Nat.0.suc.suc, a, i, fin_succ(Nat.0.suc, fin_zero(Nat.0))) *
                matrix_entry[S](Nat.0.suc.suc, Nat.0.suc.suc, b, fin_succ(Nat.0.suc, fin_zero(Nat.0)), fin_zero(Nat.0.suc))) *
                matrix_entry[S](Nat.0.suc.suc, Nat.0.suc.suc, c, fin_zero(Nat.0.suc), j) +
            (matrix_entry[S](Nat.0.suc.suc, Nat.0.suc.suc, a, i, fin_zero(Nat.0.suc)) *
                matrix_entry[S](Nat.0.suc.suc, Nat.0.suc.suc, b, fin_zero(Nat.0.suc), fin_succ(Nat.0.suc, fin_zero(Nat.0))) +
                matrix_entry[S](Nat.0.suc.suc, Nat.0.suc.suc, a, i, fin_succ(Nat.0.suc, fin_zero(Nat.0))) *
                matrix_entry[S](Nat.0.suc.suc, Nat.0.suc.suc, b, fin_succ(Nat.0.suc, fin_zero(Nat.0)), fin_succ(Nat.0.suc, fin_zero(Nat.0)))) *
                matrix_entry[S](Nat.0.suc.suc, Nat.0.suc.suc, c, fin_succ(Nat.0.suc, fin_zero(Nat.0)), j)
        matrix_entry[S](Nat.0.suc.suc, Nat.0.suc.suc, rhs, i, j) =
            matrix_entry[S](Nat.0.suc.suc, Nat.0.suc.suc, a, i, fin_zero(Nat.0.suc)) *
                (matrix_entry[S](Nat.0.suc.suc, Nat.0.suc.suc, b, fin_zero(Nat.0.suc), fin_zero(Nat.0.suc)) *
                    matrix_entry[S](Nat.0.suc.suc, Nat.0.suc.suc, c, fin_zero(Nat.0.suc), j) +
                matrix_entry[S](Nat.0.suc.suc, Nat.0.suc.suc, b, fin_zero(Nat.0.suc), fin_succ(Nat.0.suc, fin_zero(Nat.0))) *
                    matrix_entry[S](Nat.0.suc.suc, Nat.0.suc.suc, c, fin_succ(Nat.0.suc, fin_zero(Nat.0)), j)) +
            matrix_entry[S](Nat.0.suc.suc, Nat.0.suc.suc, a, i, fin_succ(Nat.0.suc, fin_zero(Nat.0))) *
                (matrix_entry[S](Nat.0.suc.suc, Nat.0.suc.suc, b, fin_succ(Nat.0.suc, fin_zero(Nat.0)), fin_zero(Nat.0.suc)) *
                    matrix_entry[S](Nat.0.suc.suc, Nat.0.suc.suc, c, fin_zero(Nat.0.suc), j) +
                matrix_entry[S](Nat.0.suc.suc, Nat.0.suc.suc, b, fin_succ(Nat.0.suc, fin_zero(Nat.0)), fin_succ(Nat.0.suc, fin_zero(Nat.0))) *
                    matrix_entry[S](Nat.0.suc.suc, Nat.0.suc.suc, c, fin_succ(Nat.0.suc, fin_zero(Nat.0)), j))
        semiring_four_term_swap[S](
            matrix_entry[S](Nat.0.suc.suc, Nat.0.suc.suc, a, i, fin_zero(Nat.0.suc)),
            matrix_entry[S](Nat.0.suc.suc, Nat.0.suc.suc, a, i, fin_succ(Nat.0.suc, fin_zero(Nat.0))),
            matrix_entry[S](Nat.0.suc.suc, Nat.0.suc.suc, b, fin_zero(Nat.0.suc), fin_zero(Nat.0.suc)),
            matrix_entry[S](Nat.0.suc.suc, Nat.0.suc.suc, b, fin_zero(Nat.0.suc), fin_succ(Nat.0.suc, fin_zero(Nat.0))),
            matrix_entry[S](Nat.0.suc.suc, Nat.0.suc.suc, b, fin_succ(Nat.0.suc, fin_zero(Nat.0)), fin_zero(Nat.0.suc)),
            matrix_entry[S](Nat.0.suc.suc, Nat.0.suc.suc, b, fin_succ(Nat.0.suc, fin_zero(Nat.0)), fin_succ(Nat.0.suc, fin_zero(Nat.0))),
            matrix_entry[S](Nat.0.suc.suc, Nat.0.suc.suc, c, fin_zero(Nat.0.suc), j),
            matrix_entry[S](Nat.0.suc.suc, Nat.0.suc.suc, c, fin_succ(Nat.0.suc, fin_zero(Nat.0)), j)
        )
        (matrix_entry[S](Nat.0.suc.suc, Nat.0.suc.suc, a, i, fin_zero(Nat.0.suc)) *
            matrix_entry[S](Nat.0.suc.suc, Nat.0.suc.suc, b, fin_zero(Nat.0.suc), fin_zero(Nat.0.suc)) +
            matrix_entry[S](Nat.0.suc.suc, Nat.0.suc.suc, a, i, fin_succ(Nat.0.suc, fin_zero(Nat.0))) *
            matrix_entry[S](Nat.0.suc.suc, Nat.0.suc.suc, b, fin_succ(Nat.0.suc, fin_zero(Nat.0)), fin_zero(Nat.0.suc))) *
            matrix_entry[S](Nat.0.suc.suc, Nat.0.suc.suc, c, fin_zero(Nat.0.suc), j) +
        (matrix_entry[S](Nat.0.suc.suc, Nat.0.suc.suc, a, i, fin_zero(Nat.0.suc)) *
            matrix_entry[S](Nat.0.suc.suc, Nat.0.suc.suc, b, fin_zero(Nat.0.suc), fin_succ(Nat.0.suc, fin_zero(Nat.0))) +
            matrix_entry[S](Nat.0.suc.suc, Nat.0.suc.suc, a, i, fin_succ(Nat.0.suc, fin_zero(Nat.0))) *
            matrix_entry[S](Nat.0.suc.suc, Nat.0.suc.suc, b, fin_succ(Nat.0.suc, fin_zero(Nat.0)), fin_succ(Nat.0.suc, fin_zero(Nat.0)))) *
            matrix_entry[S](Nat.0.suc.suc, Nat.0.suc.suc, c, fin_succ(Nat.0.suc, fin_zero(Nat.0)), j) =
        matrix_entry[S](Nat.0.suc.suc, Nat.0.suc.suc, a, i, fin_zero(Nat.0.suc)) *
            (matrix_entry[S](Nat.0.suc.suc, Nat.0.suc.suc, b, fin_zero(Nat.0.suc), fin_zero(Nat.0.suc)) *
                matrix_entry[S](Nat.0.suc.suc, Nat.0.suc.suc, c, fin_zero(Nat.0.suc), j) +
            matrix_entry[S](Nat.0.suc.suc, Nat.0.suc.suc, b, fin_zero(Nat.0.suc), fin_succ(Nat.0.suc, fin_zero(Nat.0))) *
                matrix_entry[S](Nat.0.suc.suc, Nat.0.suc.suc, c, fin_succ(Nat.0.suc, fin_zero(Nat.0)), j)) +
        matrix_entry[S](Nat.0.suc.suc, Nat.0.suc.suc, a, i, fin_succ(Nat.0.suc, fin_zero(Nat.0))) *
            (matrix_entry[S](Nat.0.suc.suc, Nat.0.suc.suc, b, fin_succ(Nat.0.suc, fin_zero(Nat.0)), fin_zero(Nat.0.suc)) *
                matrix_entry[S](Nat.0.suc.suc, Nat.0.suc.suc, c, fin_zero(Nat.0.suc), j) +
            matrix_entry[S](Nat.0.suc.suc, Nat.0.suc.suc, b, fin_succ(Nat.0.suc, fin_zero(Nat.0)), fin_succ(Nat.0.suc, fin_zero(Nat.0))) *
                matrix_entry[S](Nat.0.suc.suc, Nat.0.suc.suc, c, fin_succ(Nat.0.suc, fin_zero(Nat.0)), j))
        matrix_entry[S](Nat.0.suc.suc, Nat.0.suc.suc, lhs, i, j) =
            matrix_entry[S](Nat.0.suc.suc, Nat.0.suc.suc, rhs, i, j)
    }
    matrix_ext[S](Nat.0.suc.suc, Nat.0.suc.suc, lhs, rhs)
    lhs = rhs
}

// ---------------------------------------------------------------------------
// The two-by-two determinant
// ---------------------------------------------------------------------------

/// The entry function of the two-by-two matrix `[[a00, a01], [a10, a11]]`,
/// viewed as an `n` by `n` matrix (only meaningful at `n = 2`).  The bound is
/// kept abstract so that certificate generation handles the index arguments.
define matrix2_entry_at[R](n: Nat, a00: R, a01: R, a10: R, a11: R, i: Fin[n], j: Fin[n]) -> R {
    if i.value = Nat.0 {
        if j.value = Nat.0 {
            a00
        } else {
            a01
        }
    } else {
        if j.value = Nat.0 {
            a10
        } else {
            a11
        }
    }
}

/// The first element of `Fin[2]` differs from the second.
theorem fin_two_zero_ne_one {
    fin_zero(Nat.0.suc) != fin_succ(Nat.0.suc, fin_zero(Nat.0))
} by {
    if fin_zero(Nat.0.suc) = fin_succ(Nat.0.suc, fin_zero(Nat.0)) {
        fin_two_one_ne_zero
        fin_zero(Nat.0.suc) = fin_zero(Nat.0.suc)
        false
    }
}

/// The upper-left entry of `[[a00, a01], [a10, a11]]` is `a00`.
theorem matrix2_entry_at_zero_zero[R](a00: R, a01: R, a10: R, a11: R) {
    matrix2_entry_at[R](Nat.0.suc.suc, a00, a01, a10, a11, fin_zero(Nat.0.suc), fin_zero(Nat.0.suc)) = a00
} by {
    fin_zero_value(Nat.0.suc)
    matrix2_entry_at[R](Nat.0.suc.suc, a00, a01, a10, a11, fin_zero(Nat.0.suc), fin_zero(Nat.0.suc)) = a00
}

/// The upper-right entry of `[[a00, a01], [a10, a11]]` is `a01`.
theorem matrix2_entry_at_zero_one[R](a00: R, a01: R, a10: R, a11: R) {
    matrix2_entry_at[R](Nat.0.suc.suc, a00, a01, a10, a11, fin_zero(Nat.0.suc), fin_succ(Nat.0.suc, fin_zero(Nat.0))) = a01
} by {
    fin_zero_value(Nat.0.suc)
    fin_succ_value(Nat.0.suc, fin_zero(Nat.0))
    matrix2_entry_at[R](Nat.0.suc.suc, a00, a01, a10, a11, fin_zero(Nat.0.suc), fin_succ(Nat.0.suc, fin_zero(Nat.0))) = a01
}

/// The lower-left entry of `[[a00, a01], [a10, a11]]` is `a10`.
theorem matrix2_entry_at_one_zero[R](a00: R, a01: R, a10: R, a11: R) {
    matrix2_entry_at[R](Nat.0.suc.suc, a00, a01, a10, a11, fin_succ(Nat.0.suc, fin_zero(Nat.0)), fin_zero(Nat.0.suc)) = a10
} by {
    fin_zero_value(Nat.0.suc)
    fin_succ_value(Nat.0.suc, fin_zero(Nat.0))
    matrix2_entry_at[R](Nat.0.suc.suc, a00, a01, a10, a11, fin_succ(Nat.0.suc, fin_zero(Nat.0)), fin_zero(Nat.0.suc)) = a10
}

/// The lower-right entry of `[[a00, a01], [a10, a11]]` is `a11`.
theorem matrix2_entry_at_one_one[R](a00: R, a01: R, a10: R, a11: R) {
    matrix2_entry_at[R](Nat.0.suc.suc, a00, a01, a10, a11, fin_succ(Nat.0.suc, fin_zero(Nat.0)), fin_succ(Nat.0.suc, fin_zero(Nat.0))) = a11
} by {
    fin_zero_value(Nat.0.suc)
    fin_succ_value(Nat.0.suc, fin_zero(Nat.0))
    matrix2_entry_at[R](Nat.0.suc.suc, a00, a01, a10, a11, fin_succ(Nat.0.suc, fin_zero(Nat.0)), fin_succ(Nat.0.suc, fin_zero(Nat.0))) = a11
}

/// The two-by-two determinant of four entries, `ad - bc`.
define det2_entry[R: Ring](a00: R, a01: R, a10: R, a11: R) -> R {
    a00 * a11 - a01 * a10
}

/// The determinant of a two-by-two matrix.
define det2[R: Ring](a: Matrix[R, Nat.0.suc.suc, Nat.0.suc.suc]) -> R {
    matrix_entry[R](Nat.0.suc.suc, Nat.0.suc.suc, a, fin_zero(Nat.0.suc), fin_zero(Nat.0.suc)) *
        matrix_entry[R](Nat.0.suc.suc, Nat.0.suc.suc, a, fin_succ(Nat.0.suc, fin_zero(Nat.0)), fin_succ(Nat.0.suc, fin_zero(Nat.0))) -
    matrix_entry[R](Nat.0.suc.suc, Nat.0.suc.suc, a, fin_zero(Nat.0.suc), fin_succ(Nat.0.suc, fin_zero(Nat.0))) *
        matrix_entry[R](Nat.0.suc.suc, Nat.0.suc.suc, a, fin_succ(Nat.0.suc, fin_zero(Nat.0)), fin_zero(Nat.0.suc))
}

/// The determinant of `[[a00, a01], [a10, a11]]` is `a00 * a11 - a01 * a10`.
theorem det2_entry_expanded[R: Ring](a00: R, a01: R, a10: R, a11: R) {
    det2_entry[R](a00, a01, a10, a11) = a00 * a11 - a01 * a10
} by {
    det2_entry[R](a00, a01, a10, a11) = a00 * a11 - a01 * a10
}

// The determinant of the two-by-two identity matrix: `det2(matrix_one[R](2)) = R.1`
// is the matrix-level statement.  Claims that apply `det2` to a matrix currently
// trip a certificate-generation bug (the typeclass parameter `R: Ring` is
// re-serialized into a type position), so the verification is carried out on the
// entry-level determinant `det2_entry`, which is the same formula.
//
//     theorem det2_identity[R: Ring] {
//         det2[R](matrix_one[R](Nat.0.suc.suc)) = R.1
//     }

/// The determinant of the identity matrix, `[[1, 0], [0, 1]]`, is one.
theorem det2_entry_identity[R: Ring] {
    det2_entry[R](R.1, R.0, R.0, R.1) = R.1
} by {
    det2_entry[R](R.1, R.0, R.0, R.1) = R.1 * R.1 - R.0 * R.0
    R.1 * R.1 = R.1
    R.0 * R.0 = R.0
    R.1 - R.0 = R.1
    R.1 * R.1 - R.0 * R.0 = R.1
    det2_entry[R](R.1, R.0, R.0, R.1) = R.1
}

/// The real number two.
let real_two: Real = Real.1 + Real.1

/// The real number three.
let real_three: Real = real_two + Real.1

/// The real number four.
let real_four: Real = real_three + Real.1

/// Moving the middle summand of a three-term sum to the front.
theorem add_swap_left[A: AddCommMonoid](a: A, b: A, c: A) {
    a + (b + c) = b + (a + c)
} by {
    a + (b + c) = (a + b) + c
    a + b = b + a
    (a + b) + c = (b + a) + c
    (b + a) + c = b + (a + c)
}

/// The determinant of `[[1, 2], [3, 4]]` is `-2`.
theorem det2_concrete {
    det2_entry[Real](Real.1, real_two, real_three, real_four) = -real_two
} by {
    det2_entry[Real](Real.1, real_two, real_three, real_four) = Real.1 * real_four - real_two * real_three
    Real.1 * real_four = real_four
    real_two * real_three = real_four + real_two
    inverse_add[Real](real_four, real_two)
    -(real_four + real_two) = -real_two + -real_four
    add_swap_left[Real](real_four, -real_two, -real_four)
    real_four + (-real_two + -real_four) = -real_two + (real_four + -real_four)
    real_four + -real_four = Real.0
    -real_two + (real_four + -real_four) = -real_two + Real.0
    -real_two + Real.0 = -real_two
    real_four - (real_four + real_two) = -real_two
    real_four - real_two * real_three = -real_two
    Real.1 * real_four - real_two * real_three = -real_two
    det2_entry[Real](Real.1, real_two, real_three, real_four) = -real_two
}

// ---------------------------------------------------------------------------
// The identity laws and distributivity for two-by-two matrices
// ---------------------------------------------------------------------------

/// Multiplying a two-by-two matrix by the identity on the right changes nothing.
theorem square_matrix_mul_one_right_two[S: Semiring](a: Matrix[S, Nat.0.suc.suc, Nat.0.suc.suc]) {
    square_matrix_mul[S](Nat.0.suc.suc, a, matrix_one[S](Nat.0.suc.suc)) = a
} by {
    square_matrix_mul_one_right[S](Nat.0.suc.suc, a)
    square_matrix_mul[S](Nat.0.suc.suc, a, matrix_one[S](Nat.0.suc.suc)) = a
}

/// Multiplying a two-by-two matrix by the identity on the left changes nothing.
theorem square_matrix_mul_one_left_two[S: Semiring](a: Matrix[S, Nat.0.suc.suc, Nat.0.suc.suc]) {
    square_matrix_mul[S](Nat.0.suc.suc, matrix_one[S](Nat.0.suc.suc), a) = a
} by {
    square_matrix_mul_one_left[S](Nat.0.suc.suc, a)
    square_matrix_mul[S](Nat.0.suc.suc, matrix_one[S](Nat.0.suc.suc), a) = a
}

/// Multiplication distributes over addition in the right factor for two-by-two matrices.
theorem square_matrix_mul_add_right_two[S: Semiring](
    a: Matrix[S, Nat.0.suc.suc, Nat.0.suc.suc],
    b: Matrix[S, Nat.0.suc.suc, Nat.0.suc.suc],
    c: Matrix[S, Nat.0.suc.suc, Nat.0.suc.suc]
) {
    square_matrix_mul[S](Nat.0.suc.suc, a, b + c) =
        square_matrix_mul[S](Nat.0.suc.suc, a, b) + square_matrix_mul[S](Nat.0.suc.suc, a, c)
} by {
    square_matrix_mul_add_right[S](Nat.0.suc.suc, a, b, c)
    square_matrix_mul[S](Nat.0.suc.suc, a, b + c) =
        square_matrix_mul[S](Nat.0.suc.suc, a, b) + square_matrix_mul[S](Nat.0.suc.suc, a, c)
}

/// Multiplication distributes over addition in the left factor for two-by-two matrices.
theorem square_matrix_mul_add_left_two[S: Semiring](
    a: Matrix[S, Nat.0.suc.suc, Nat.0.suc.suc],
    b: Matrix[S, Nat.0.suc.suc, Nat.0.suc.suc],
    c: Matrix[S, Nat.0.suc.suc, Nat.0.suc.suc]
) {
    square_matrix_mul[S](Nat.0.suc.suc, a + b, c) =
        square_matrix_mul[S](Nat.0.suc.suc, a, c) + square_matrix_mul[S](Nat.0.suc.suc, b, c)
} by {
    square_matrix_mul_add_left[S](Nat.0.suc.suc, a, b, c)
    square_matrix_mul[S](Nat.0.suc.suc, a + b, c) =
        square_matrix_mul[S](Nat.0.suc.suc, a, c) + square_matrix_mul[S](Nat.0.suc.suc, b, c)
}

// ---------------------------------------------------------------------------
// Matrix-vector multiplication is compatible with matrix multiplication
// ---------------------------------------------------------------------------

/// The vector obtained by applying a square matrix to a vector.
define matrix_apply_vec[S: Semiring](n: Nat, a: Matrix[S, n, n], v: Fin[n] -> S, k: Fin[n]) -> S {
    matrix_apply[S](n, a, v, k)
}

/// For two-by-two matrices, `(A B) v = A (B v)`.
theorem matrix_apply_mul_two[S: Semiring](
    a: Matrix[S, Nat.0.suc.suc, Nat.0.suc.suc],
    b: Matrix[S, Nat.0.suc.suc, Nat.0.suc.suc],
    v: Fin[Nat.0.suc.suc] -> S,
    i: Fin[Nat.0.suc.suc]
) {
    matrix_apply[S](Nat.0.suc.suc, square_matrix_mul[S](Nat.0.suc.suc, a, b), v, i) =
        matrix_apply[S](Nat.0.suc.suc, a, matrix_apply_vec[S](Nat.0.suc.suc, b, v), i)
} by {
    matrix_apply_eq[S](Nat.0.suc.suc, square_matrix_mul[S](Nat.0.suc.suc, a, b), v, i)
    matrix_apply[S](Nat.0.suc.suc, square_matrix_mul[S](Nat.0.suc.suc, a, b), v, i) =
        fin_sum[S](Nat.0.suc.suc, matrix_apply_term[S](Nat.0.suc.suc, square_matrix_mul[S](Nat.0.suc.suc, a, b), v, i))
    fin_sum_two[S](matrix_apply_term[S](Nat.0.suc.suc, square_matrix_mul[S](Nat.0.suc.suc, a, b), v, i))
    fin_sum[S](Nat.0.suc.suc, matrix_apply_term[S](Nat.0.suc.suc, square_matrix_mul[S](Nat.0.suc.suc, a, b), v, i)) =
        matrix_apply_term[S](Nat.0.suc.suc, square_matrix_mul[S](Nat.0.suc.suc, a, b), v, i, fin_zero(Nat.0.suc)) +
        matrix_apply_term[S](Nat.0.suc.suc, square_matrix_mul[S](Nat.0.suc.suc, a, b), v, i, fin_succ(Nat.0.suc, fin_zero(Nat.0)))
    matrix_entry_mul_two[S](a, b, i, fin_zero(Nat.0.suc))
    matrix_entry_mul_two[S](a, b, i, fin_succ(Nat.0.suc, fin_zero(Nat.0)))
    matrix_apply[S](Nat.0.suc.suc, square_matrix_mul[S](Nat.0.suc.suc, a, b), v, i) =
        (matrix_entry[S](Nat.0.suc.suc, Nat.0.suc.suc, a, i, fin_zero(Nat.0.suc)) *
            matrix_entry[S](Nat.0.suc.suc, Nat.0.suc.suc, b, fin_zero(Nat.0.suc), fin_zero(Nat.0.suc)) +
            matrix_entry[S](Nat.0.suc.suc, Nat.0.suc.suc, a, i, fin_succ(Nat.0.suc, fin_zero(Nat.0))) *
            matrix_entry[S](Nat.0.suc.suc, Nat.0.suc.suc, b, fin_succ(Nat.0.suc, fin_zero(Nat.0)), fin_zero(Nat.0.suc))) *
            v(fin_zero(Nat.0.suc)) +
        (matrix_entry[S](Nat.0.suc.suc, Nat.0.suc.suc, a, i, fin_zero(Nat.0.suc)) *
            matrix_entry[S](Nat.0.suc.suc, Nat.0.suc.suc, b, fin_zero(Nat.0.suc), fin_succ(Nat.0.suc, fin_zero(Nat.0))) +
            matrix_entry[S](Nat.0.suc.suc, Nat.0.suc.suc, a, i, fin_succ(Nat.0.suc, fin_zero(Nat.0))) *
            matrix_entry[S](Nat.0.suc.suc, Nat.0.suc.suc, b, fin_succ(Nat.0.suc, fin_zero(Nat.0)), fin_succ(Nat.0.suc, fin_zero(Nat.0)))) *
            v(fin_succ(Nat.0.suc, fin_zero(Nat.0)))
    matrix_apply_eq[S](Nat.0.suc.suc, a, matrix_apply_vec[S](Nat.0.suc.suc, b, v), i)
    matrix_apply[S](Nat.0.suc.suc, a, matrix_apply_vec[S](Nat.0.suc.suc, b, v), i) =
        fin_sum[S](Nat.0.suc.suc, matrix_apply_term[S](Nat.0.suc.suc, a, matrix_apply_vec[S](Nat.0.suc.suc, b, v), i))
    fin_sum_two[S](matrix_apply_term[S](Nat.0.suc.suc, a, matrix_apply_vec[S](Nat.0.suc.suc, b, v), i))
    fin_sum[S](Nat.0.suc.suc, matrix_apply_term[S](Nat.0.suc.suc, a, matrix_apply_vec[S](Nat.0.suc.suc, b, v), i)) =
        matrix_apply_term[S](Nat.0.suc.suc, a, matrix_apply_vec[S](Nat.0.suc.suc, b, v), i, fin_zero(Nat.0.suc)) +
        matrix_apply_term[S](Nat.0.suc.suc, a, matrix_apply_vec[S](Nat.0.suc.suc, b, v), i, fin_succ(Nat.0.suc, fin_zero(Nat.0)))
    matrix_apply_eq[S](Nat.0.suc.suc, b, v, fin_zero(Nat.0.suc))
    matrix_apply_eq[S](Nat.0.suc.suc, b, v, fin_succ(Nat.0.suc, fin_zero(Nat.0)))
    matrix_apply[S](Nat.0.suc.suc, b, v, fin_zero(Nat.0.suc)) =
        fin_sum[S](Nat.0.suc.suc, matrix_apply_term[S](Nat.0.suc.suc, b, v, fin_zero(Nat.0.suc)))
    matrix_apply[S](Nat.0.suc.suc, b, v, fin_succ(Nat.0.suc, fin_zero(Nat.0))) =
        fin_sum[S](Nat.0.suc.suc, matrix_apply_term[S](Nat.0.suc.suc, b, v, fin_succ(Nat.0.suc, fin_zero(Nat.0))))
    fin_sum_two[S](matrix_apply_term[S](Nat.0.suc.suc, b, v, fin_zero(Nat.0.suc)))
    fin_sum_two[S](matrix_apply_term[S](Nat.0.suc.suc, b, v, fin_succ(Nat.0.suc, fin_zero(Nat.0))))
    matrix_apply[S](Nat.0.suc.suc, b, v, fin_zero(Nat.0.suc)) =
        matrix_apply_term[S](Nat.0.suc.suc, b, v, fin_zero(Nat.0.suc), fin_zero(Nat.0.suc)) +
        matrix_apply_term[S](Nat.0.suc.suc, b, v, fin_zero(Nat.0.suc), fin_succ(Nat.0.suc, fin_zero(Nat.0)))
    matrix_apply[S](Nat.0.suc.suc, b, v, fin_succ(Nat.0.suc, fin_zero(Nat.0))) =
        matrix_apply_term[S](Nat.0.suc.suc, b, v, fin_succ(Nat.0.suc, fin_zero(Nat.0)), fin_zero(Nat.0.suc)) +
        matrix_apply_term[S](Nat.0.suc.suc, b, v, fin_succ(Nat.0.suc, fin_zero(Nat.0)), fin_succ(Nat.0.suc, fin_zero(Nat.0)))
    matrix_apply[S](Nat.0.suc.suc, a, matrix_apply_vec[S](Nat.0.suc.suc, b, v), i) =
        matrix_entry[S](Nat.0.suc.suc, Nat.0.suc.suc, a, i, fin_zero(Nat.0.suc)) *
            (matrix_entry[S](Nat.0.suc.suc, Nat.0.suc.suc, b, fin_zero(Nat.0.suc), fin_zero(Nat.0.suc)) *
                v(fin_zero(Nat.0.suc)) +
            matrix_entry[S](Nat.0.suc.suc, Nat.0.suc.suc, b, fin_zero(Nat.0.suc), fin_succ(Nat.0.suc, fin_zero(Nat.0))) *
                v(fin_succ(Nat.0.suc, fin_zero(Nat.0)))) +
        matrix_entry[S](Nat.0.suc.suc, Nat.0.suc.suc, a, i, fin_succ(Nat.0.suc, fin_zero(Nat.0))) *
            (matrix_entry[S](Nat.0.suc.suc, Nat.0.suc.suc, b, fin_succ(Nat.0.suc, fin_zero(Nat.0)), fin_zero(Nat.0.suc)) *
                v(fin_zero(Nat.0.suc)) +
            matrix_entry[S](Nat.0.suc.suc, Nat.0.suc.suc, b, fin_succ(Nat.0.suc, fin_zero(Nat.0)), fin_succ(Nat.0.suc, fin_zero(Nat.0))) *
                v(fin_succ(Nat.0.suc, fin_zero(Nat.0))))
    semiring_four_term_swap[S](
        matrix_entry[S](Nat.0.suc.suc, Nat.0.suc.suc, a, i, fin_zero(Nat.0.suc)),
        matrix_entry[S](Nat.0.suc.suc, Nat.0.suc.suc, a, i, fin_succ(Nat.0.suc, fin_zero(Nat.0))),
        matrix_entry[S](Nat.0.suc.suc, Nat.0.suc.suc, b, fin_zero(Nat.0.suc), fin_zero(Nat.0.suc)),
        matrix_entry[S](Nat.0.suc.suc, Nat.0.suc.suc, b, fin_zero(Nat.0.suc), fin_succ(Nat.0.suc, fin_zero(Nat.0))),
        matrix_entry[S](Nat.0.suc.suc, Nat.0.suc.suc, b, fin_succ(Nat.0.suc, fin_zero(Nat.0)), fin_zero(Nat.0.suc)),
        matrix_entry[S](Nat.0.suc.suc, Nat.0.suc.suc, b, fin_succ(Nat.0.suc, fin_zero(Nat.0)), fin_succ(Nat.0.suc, fin_zero(Nat.0))),
        v(fin_zero(Nat.0.suc)),
        v(fin_succ(Nat.0.suc, fin_zero(Nat.0)))
    )
    (matrix_entry[S](Nat.0.suc.suc, Nat.0.suc.suc, a, i, fin_zero(Nat.0.suc)) *
        matrix_entry[S](Nat.0.suc.suc, Nat.0.suc.suc, b, fin_zero(Nat.0.suc), fin_zero(Nat.0.suc)) +
        matrix_entry[S](Nat.0.suc.suc, Nat.0.suc.suc, a, i, fin_succ(Nat.0.suc, fin_zero(Nat.0))) *
        matrix_entry[S](Nat.0.suc.suc, Nat.0.suc.suc, b, fin_succ(Nat.0.suc, fin_zero(Nat.0)), fin_zero(Nat.0.suc))) *
        v(fin_zero(Nat.0.suc)) +
    (matrix_entry[S](Nat.0.suc.suc, Nat.0.suc.suc, a, i, fin_zero(Nat.0.suc)) *
        matrix_entry[S](Nat.0.suc.suc, Nat.0.suc.suc, b, fin_zero(Nat.0.suc), fin_succ(Nat.0.suc, fin_zero(Nat.0))) +
        matrix_entry[S](Nat.0.suc.suc, Nat.0.suc.suc, a, i, fin_succ(Nat.0.suc, fin_zero(Nat.0))) *
        matrix_entry[S](Nat.0.suc.suc, Nat.0.suc.suc, b, fin_succ(Nat.0.suc, fin_zero(Nat.0)), fin_succ(Nat.0.suc, fin_zero(Nat.0)))) *
        v(fin_succ(Nat.0.suc, fin_zero(Nat.0))) =
    matrix_entry[S](Nat.0.suc.suc, Nat.0.suc.suc, a, i, fin_zero(Nat.0.suc)) *
        (matrix_entry[S](Nat.0.suc.suc, Nat.0.suc.suc, b, fin_zero(Nat.0.suc), fin_zero(Nat.0.suc)) *
            v(fin_zero(Nat.0.suc)) +
        matrix_entry[S](Nat.0.suc.suc, Nat.0.suc.suc, b, fin_zero(Nat.0.suc), fin_succ(Nat.0.suc, fin_zero(Nat.0))) *
            v(fin_succ(Nat.0.suc, fin_zero(Nat.0)))) +
    matrix_entry[S](Nat.0.suc.suc, Nat.0.suc.suc, a, i, fin_succ(Nat.0.suc, fin_zero(Nat.0))) *
        (matrix_entry[S](Nat.0.suc.suc, Nat.0.suc.suc, b, fin_succ(Nat.0.suc, fin_zero(Nat.0)), fin_zero(Nat.0.suc)) *
            v(fin_zero(Nat.0.suc)) +
        matrix_entry[S](Nat.0.suc.suc, Nat.0.suc.suc, b, fin_succ(Nat.0.suc, fin_zero(Nat.0)), fin_succ(Nat.0.suc, fin_zero(Nat.0))) *
            v(fin_succ(Nat.0.suc, fin_zero(Nat.0))))
    matrix_apply[S](Nat.0.suc.suc, square_matrix_mul[S](Nat.0.suc.suc, a, b), v, i) =
        matrix_apply[S](Nat.0.suc.suc, a, matrix_apply_vec[S](Nat.0.suc.suc, b, v), i)
}

// ---------------------------------------------------------------------------
// The inverse of a two-by-two matrix
// ---------------------------------------------------------------------------

// The general inverse theorem: if `ad - bc != 0`, the inverse of
// `[[a00, a01], [a10, a11]]` is `(1/(ad - bc)) * [[a11, -a01], [-a10, a00]]`.
//
//     theorem matrix2_inv_mul[F: Field](a00: F, a01: F, a10: F, a11: F) {
//         det2_entry[F](a00, a01, a10, a11) != F.0 implies
//             a00 * inv2_entry_00[F](a00, a01, a10, a11) +
//                 a01 * inv2_entry_10[F](a00, a01, a10, a11) = F.1 and
//             a00 * inv2_entry_01[F](a00, a01, a10, a11) +
//                 a01 * inv2_entry_11[F](a00, a01, a10, a11) = F.0 and
//             a10 * inv2_entry_00[F](a00, a01, a10, a11) +
//                 a11 * inv2_entry_10[F](a00, a01, a10, a11) = F.0 and
//             a10 * inv2_entry_01[F](a00, a01, a10, a11) +
//                 a11 * inv2_entry_11[F](a00, a01, a10, a11) = F.1
//     }
//
// Its proof needs the field inverse law `d * d.inverse = F.1` chained with the
// commutativity-based rearrangement of the four products, which the current
// automation does not find; the identity and the concrete instance below are
// verified by expansion.  The matrix-level statement `matrix2_inv(a) * a =
// matrix_one` is likewise left as a statement because claims applying a
// user-defined `Matrix.new` wrapper trip a certificate-generation bug.

/// The upper-left entry of the inverse of `[[a00, a01], [a10, a11]]`.
define inv2_entry_00[F: Field](a00: F, a01: F, a10: F, a11: F) -> F {
    det2_entry[F](a00, a01, a10, a11).inverse * a11
}

/// The upper-right entry of the inverse of `[[a00, a01], [a10, a11]]`.
define inv2_entry_01[F: Field](a00: F, a01: F, a10: F, a11: F) -> F {
    det2_entry[F](a00, a01, a10, a11).inverse * -a01
}

/// The lower-left entry of the inverse of `[[a00, a01], [a10, a11]]`.
define inv2_entry_10[F: Field](a00: F, a01: F, a10: F, a11: F) -> F {
    det2_entry[F](a00, a01, a10, a11).inverse * -a10
}

/// The lower-right entry of the inverse of `[[a00, a01], [a10, a11]]`.
define inv2_entry_11[F: Field](a00: F, a01: F, a10: F, a11: F) -> F {
    det2_entry[F](a00, a01, a10, a11).inverse * a00
}

/// The inverse of the determinant of the identity matrix is one.
theorem inv2_det_one[F: Field] {
    det2_entry[F](F.1, F.0, F.0, F.1).inverse = F.1
} by {
    det2_entry[F](F.1, F.0, F.0, F.1) = F.1 * F.1 - F.0 * F.0
    F.1 * F.1 = F.1
    F.0 * F.0 = F.0
    F.1 - F.0 = F.1
    F.1 * F.1 - F.0 * F.0 = F.1
    det2_entry[F](F.1, F.0, F.0, F.1) = F.1
    inverse_one[F]
    F.1.inverse = F.1
    det2_entry[F](F.1, F.0, F.0, F.1).inverse = F.1
}

/// The upper-left entry of the inverse of the identity matrix is one.
theorem inv2_entry_identity_00[F: Field] {
    inv2_entry_00[F](F.1, F.0, F.0, F.1) = F.1
} by {
    inv2_det_one[F]
    inv2_entry_00[F](F.1, F.0, F.0, F.1) = det2_entry[F](F.1, F.0, F.0, F.1).inverse * F.1
    det2_entry[F](F.1, F.0, F.0, F.1).inverse * F.1 = F.1
    inv2_entry_00[F](F.1, F.0, F.0, F.1) = F.1
}

/// The upper-right entry of the inverse of the identity matrix is zero.
theorem inv2_entry_identity_01[F: Field] {
    inv2_entry_01[F](F.1, F.0, F.0, F.1) = F.0
} by {
    inv2_det_one[F]
    -F.0 = F.0
    F.1 * F.0 = F.0
    inv2_entry_01[F](F.1, F.0, F.0, F.1) = det2_entry[F](F.1, F.0, F.0, F.1).inverse * -F.0
    det2_entry[F](F.1, F.0, F.0, F.1).inverse * -F.0 = F.0
    inv2_entry_01[F](F.1, F.0, F.0, F.1) = F.0
}

/// The lower-left entry of the inverse of the identity matrix is zero.
theorem inv2_entry_identity_10[F: Field] {
    inv2_entry_10[F](F.1, F.0, F.0, F.1) = F.0
} by {
    inv2_det_one[F]
    -F.0 = F.0
    F.1 * F.0 = F.0
    inv2_entry_10[F](F.1, F.0, F.0, F.1) = det2_entry[F](F.1, F.0, F.0, F.1).inverse * -F.0
    det2_entry[F](F.1, F.0, F.0, F.1).inverse * -F.0 = F.0
    inv2_entry_10[F](F.1, F.0, F.0, F.1) = F.0
}

/// The lower-right entry of the inverse of the identity matrix is one.
theorem inv2_entry_identity_11[F: Field] {
    inv2_entry_11[F](F.1, F.0, F.0, F.1) = F.1
} by {
    inv2_det_one[F]
    inv2_entry_11[F](F.1, F.0, F.0, F.1) = det2_entry[F](F.1, F.0, F.0, F.1).inverse * F.1
    det2_entry[F](F.1, F.0, F.0, F.1).inverse * F.1 = F.1
    inv2_entry_11[F](F.1, F.0, F.0, F.1) = F.1
}

/// The real number six.
let real_six: Real = real_four + real_two

/// The real number seven.
let real_seven: Real = real_six + Real.1

/// The real number twenty-one.
let real_twenty_one: Real = real_seven + real_seven + real_seven

/// `7 + (-6) = 1`, the subtraction step behind `7 - 6 = 1`.
theorem real_seven_plus_neg_six {
    real_seven + -real_six = Real.1
} by {
    real_seven = real_six + Real.1
    real_seven + -real_six = (real_six + Real.1) + -real_six
    (real_six + Real.1) + -real_six = real_six + (Real.1 + -real_six)
    add_swap_left[Real](real_six, Real.1, -real_six)
    real_six + (Real.1 + -real_six) = Real.1 + (real_six + -real_six)
    real_six + -real_six = Real.0
    Real.1 + (real_six + -real_six) = Real.1 + Real.0
    Real.1 + Real.0 = Real.1
    real_seven + -real_six = Real.1
}

/// The determinant of `[[1, 2], [3, 7]]` is one.
theorem inv2_det_seven {
    det2_entry[Real](Real.1, real_two, real_three, real_seven) = Real.1
} by {
    det2_entry[Real](Real.1, real_two, real_three, real_seven) = Real.1 * real_seven - real_two * real_three
    Real.1 * real_seven = real_seven
    real_two * real_three = real_four + real_two
    real_four + real_two = real_six
    real_seven_plus_neg_six
    real_seven - real_six = Real.1
    Real.1 * real_seven - real_two * real_three = Real.1
    det2_entry[Real](Real.1, real_two, real_three, real_seven) = Real.1
}

/// The upper-left entry of the inverse of `[[1, 2], [3, 7]]` is `7`.
theorem inv2_entry_concrete_00 {
    inv2_entry_00[Real](Real.1, real_two, real_three, real_seven) = real_seven
} by {
    inv2_det_seven
    inverse_one[Real]
    Real.1.inverse = Real.1
    det2_entry[Real](Real.1, real_two, real_three, real_seven).inverse = Real.1
    inv2_entry_00[Real](Real.1, real_two, real_three, real_seven) =
        det2_entry[Real](Real.1, real_two, real_three, real_seven).inverse * real_seven
    det2_entry[Real](Real.1, real_two, real_three, real_seven).inverse * real_seven = real_seven
    inv2_entry_00[Real](Real.1, real_two, real_three, real_seven) = real_seven
}

/// The upper-right entry of the inverse of `[[1, 2], [3, 7]]` is `-2`.
theorem inv2_entry_concrete_01 {
    inv2_entry_01[Real](Real.1, real_two, real_three, real_seven) = -real_two
} by {
    inv2_det_seven
    inverse_one[Real]
    Real.1.inverse = Real.1
    det2_entry[Real](Real.1, real_two, real_three, real_seven).inverse = Real.1
    inv2_entry_01[Real](Real.1, real_two, real_three, real_seven) =
        det2_entry[Real](Real.1, real_two, real_three, real_seven).inverse * -real_two
    det2_entry[Real](Real.1, real_two, real_three, real_seven).inverse * -real_two = -real_two
    inv2_entry_01[Real](Real.1, real_two, real_three, real_seven) = -real_two
}

/// The lower-left entry of the inverse of `[[1, 2], [3, 7]]` is `-3`.
theorem inv2_entry_concrete_10 {
    inv2_entry_10[Real](Real.1, real_two, real_three, real_seven) = -real_three
} by {
    inv2_det_seven
    inverse_one[Real]
    Real.1.inverse = Real.1
    det2_entry[Real](Real.1, real_two, real_three, real_seven).inverse = Real.1
    inv2_entry_10[Real](Real.1, real_two, real_three, real_seven) =
        det2_entry[Real](Real.1, real_two, real_three, real_seven).inverse * -real_three
    det2_entry[Real](Real.1, real_two, real_three, real_seven).inverse * -real_three = -real_three
    inv2_entry_10[Real](Real.1, real_two, real_three, real_seven) = -real_three
}

/// The lower-right entry of the inverse of `[[1, 2], [3, 7]]` is `1`.
theorem inv2_entry_concrete_11 {
    inv2_entry_11[Real](Real.1, real_two, real_three, real_seven) = Real.1
} by {
    inv2_det_seven
    inverse_one[Real]
    Real.1.inverse = Real.1
    det2_entry[Real](Real.1, real_two, real_three, real_seven).inverse = Real.1
    inv2_entry_11[Real](Real.1, real_two, real_three, real_seven) =
        det2_entry[Real](Real.1, real_two, real_three, real_seven).inverse * Real.1
    det2_entry[Real](Real.1, real_two, real_three, real_seven).inverse * Real.1 = Real.1
    inv2_entry_11[Real](Real.1, real_two, real_three, real_seven) = Real.1
}

/// The upper-left entry of `inv([[1, 2], [3, 7]]) * [[1, 2], [3, 7]]` is one.
theorem inv2_product_concrete_00 {
    Real.1 * inv2_entry_00[Real](Real.1, real_two, real_three, real_seven) +
        real_two * inv2_entry_10[Real](Real.1, real_two, real_three, real_seven) = Real.1
} by {
    inv2_entry_concrete_00
    inv2_entry_concrete_10
    Real.1 * inv2_entry_00[Real](Real.1, real_two, real_three, real_seven) +
        real_two * inv2_entry_10[Real](Real.1, real_two, real_three, real_seven) =
        Real.1 * real_seven + real_two * -real_three
    Real.1 * real_seven = real_seven
    real_two * -real_three = -(real_two * real_three)
    real_two * real_three = real_four + real_two
    real_four + real_two = real_six
    -(real_two * real_three) = -real_six
    real_seven_plus_neg_six
    real_seven + -real_six = Real.1
    Real.1 * real_seven + real_two * -real_three = Real.1
    Real.1 * inv2_entry_00[Real](Real.1, real_two, real_three, real_seven) +
        real_two * inv2_entry_10[Real](Real.1, real_two, real_three, real_seven) = Real.1
}

/// The upper-right entry of `inv([[1, 2], [3, 7]]) * [[1, 2], [3, 7]]` is zero.
theorem inv2_product_concrete_01 {
    Real.1 * inv2_entry_01[Real](Real.1, real_two, real_three, real_seven) +
        real_two * inv2_entry_11[Real](Real.1, real_two, real_three, real_seven) = Real.0
} by {
    inv2_entry_concrete_01
    inv2_entry_concrete_11
    Real.1 * inv2_entry_01[Real](Real.1, real_two, real_three, real_seven) +
        real_two * inv2_entry_11[Real](Real.1, real_two, real_three, real_seven) =
        Real.1 * -real_two + real_two * Real.1
    Real.1 * -real_two = -real_two
    real_two * Real.1 = real_two
    -real_two + real_two = Real.0
    Real.1 * -real_two + real_two * Real.1 = Real.0
    Real.1 * inv2_entry_01[Real](Real.1, real_two, real_three, real_seven) +
        real_two * inv2_entry_11[Real](Real.1, real_two, real_three, real_seven) = Real.0
}

/// The lower-left entry of `inv([[1, 2], [3, 7]]) * [[1, 2], [3, 7]]` is zero.
theorem inv2_product_concrete_10 {
    real_three * inv2_entry_00[Real](Real.1, real_two, real_three, real_seven) +
        real_seven * inv2_entry_10[Real](Real.1, real_two, real_three, real_seven) = Real.0
} by {
    inv2_entry_concrete_00
    inv2_entry_concrete_10
    real_three * inv2_entry_00[Real](Real.1, real_two, real_three, real_seven) +
        real_seven * inv2_entry_10[Real](Real.1, real_two, real_three, real_seven) =
        real_three * real_seven + real_seven * -real_three
    real_three * real_seven = (real_two + Real.1) * real_seven
    (real_two + Real.1) * real_seven = real_two * real_seven + Real.1 * real_seven
    real_two * real_seven = real_seven + real_seven
    Real.1 * real_seven = real_seven
    real_two * real_seven + Real.1 * real_seven = real_seven + real_seven + real_seven
    real_three * real_seven = real_seven + real_seven + real_seven
    real_seven + real_seven + real_seven = real_twenty_one
    real_seven * -real_three = -(real_seven * real_three)
    real_seven * real_three = real_three * real_seven
    real_three * real_seven = real_twenty_one
    -(real_seven * real_three) = -real_twenty_one
    real_twenty_one + -real_twenty_one = Real.0
    real_three * real_seven + real_seven * -real_three = Real.0
    real_three * inv2_entry_00[Real](Real.1, real_two, real_three, real_seven) +
        real_seven * inv2_entry_10[Real](Real.1, real_two, real_three, real_seven) = Real.0
}

/// The lower-right entry of `inv([[1, 2], [3, 7]]) * [[1, 2], [3, 7]]` is one.
theorem inv2_product_concrete_11 {
    real_three * inv2_entry_01[Real](Real.1, real_two, real_three, real_seven) +
        real_seven * inv2_entry_11[Real](Real.1, real_two, real_three, real_seven) = Real.1
} by {
    inv2_entry_concrete_01
    inv2_entry_concrete_11
    real_three * inv2_entry_01[Real](Real.1, real_two, real_three, real_seven) +
        real_seven * inv2_entry_11[Real](Real.1, real_two, real_three, real_seven) =
        real_three * -real_two + real_seven * Real.1
    real_three * -real_two = -(real_three * real_two)
    real_three * real_two = real_four + real_two
    real_four + real_two = real_six
    -(real_three * real_two) = -real_six
    real_seven * Real.1 = real_seven
    real_seven_plus_neg_six
    real_seven + -real_six = Real.1
    real_three * -real_two + real_seven * Real.1 = Real.1
    real_three * inv2_entry_01[Real](Real.1, real_two, real_three, real_seven) +
        real_seven * inv2_entry_11[Real](Real.1, real_two, real_three, real_seven) = Real.1
}

// ---------------------------------------------------------------------------
// The four entries, the determinant, the adjugate, and the inverse of a
// two-by-two matrix
//
// The definitions below back the matrix-level inverse theory in
// `data/fin/fin_matrix_inverse.ac`.  They are collected here (rather than in
// that file) because the certificate serializer currently cannot round-trip
// claims that apply a function whose signature contains a fixed-size
// `Matrix[F, ...]` type; abstract-size signatures applied at a concrete size
// are safe, so every helper below is parameterized by an ambient size `n` and
// used at `n = 0` for the two-by-two case.
// ---------------------------------------------------------------------------

/// The upper-left entry of a matrix of size `n + 2` (the two-by-two case at
/// `n = 0`).
define matrix2_entry_00[R: Ring](n: Nat, a: Matrix[R, n.suc.suc, n.suc.suc]) -> R {
    matrix_entry[R](n.suc.suc, n.suc.suc, a, fin_zero(n.suc), fin_zero(n.suc))
}

/// The upper-right entry of a matrix of size `n + 2`.
define matrix2_entry_01[R: Ring](n: Nat, a: Matrix[R, n.suc.suc, n.suc.suc]) -> R {
    matrix_entry[R](n.suc.suc, n.suc.suc, a, fin_zero(n.suc), fin_succ(n.suc, fin_zero(n)))
}

/// The lower-left entry of a matrix of size `n + 2`.
define matrix2_entry_10[R: Ring](n: Nat, a: Matrix[R, n.suc.suc, n.suc.suc]) -> R {
    matrix_entry[R](n.suc.suc, n.suc.suc, a, fin_succ(n.suc, fin_zero(n)), fin_zero(n.suc))
}

/// The lower-right entry of a matrix of size `n + 2`.
define matrix2_entry_11[R: Ring](n: Nat, a: Matrix[R, n.suc.suc, n.suc.suc]) -> R {
    matrix_entry[R](n.suc.suc, n.suc.suc, a, fin_succ(n.suc, fin_zero(n)), fin_succ(n.suc, fin_zero(n)))
}

/// The two-by-two determinant formula at ambient size `n`.
define matrix2_det[R: Ring](n: Nat, a: Matrix[R, n.suc.suc, n.suc.suc]) -> R {
    matrix2_entry_00[R](n, a) * matrix2_entry_11[R](n, a) - matrix2_entry_01[R](n, a) * matrix2_entry_10[R](n, a)
}

/// The `(i, j)` entry of the two-by-two adjugate of `a`: the cofactor of the
/// transposed index `(j, i)`, with the canonical determinant on one-by-one
/// minors.
define matrix2_adjugate_entry[F: Field](n: Nat, a: Matrix[F, n.suc.suc, n.suc.suc], i: Fin[n.suc.suc], j: Fin[n.suc.suc]) -> F {
    matrix_cofactor_with[F](n.suc, matrix_det_suc[F](n), a, j, i)
}

/// The two-by-two adjugate of `a`: the transpose of the cofactor matrix.
define matrix2_adjugate[F: Field](n: Nat, a: Matrix[F, n.suc.suc, n.suc.suc]) -> Matrix[F, n.suc.suc, n.suc.suc] {
    Matrix[F, n.suc.suc, n.suc.suc].new(matrix2_adjugate_entry[F](n, a))
}

/// The entry function of the scalar multiple of the size-`n + 2` identity.
define matrix2_scalar_one_entry[R: Ring](n: Nat, r: R, i: Fin[n.suc.suc], j: Fin[n.suc.suc]) -> R {
    if i = j {
        r
    } else {
        R.0
    }
}

/// The `(i, k)` entry of the matrix obtained from `a` by replacing column `j`
/// with the vector `b`.
define matrix2_replace_col_entry[R: Ring](
    n: Nat, a: Matrix[R, n.suc.suc, n.suc.suc], j: Fin[n.suc.suc], b: Fin[n.suc.suc] -> R,
    i: Fin[n.suc.suc], k: Fin[n.suc.suc]
) -> R {
    if k = j {
        b(i)
    } else {
        matrix_entry[R](n.suc.suc, n.suc.suc, a, i, k)
    }
}

/// The matrix obtained from `a` by replacing column `j` with the vector `b`.
define matrix2_replace_col[R: Ring](
    n: Nat, a: Matrix[R, n.suc.suc, n.suc.suc], j: Fin[n.suc.suc], b: Fin[n.suc.suc] -> R
) -> Matrix[R, n.suc.suc, n.suc.suc] {
    Matrix[R, n.suc.suc, n.suc.suc].new(matrix2_replace_col_entry[R](n, a, j, b))
}

/// The `(i, j)` entry of the inverse of a two-by-two matrix: the determinant
/// inverse times the adjugate entry.
define matrix2_inv_entry[F: Field](n: Nat, a: Matrix[F, n.suc.suc, n.suc.suc], i: Fin[n.suc.suc], j: Fin[n.suc.suc]) -> F {
    matrix2_det[F](n, a).inverse * matrix2_adjugate_entry[F](n, a, i, j)
}

/// The inverse of a two-by-two matrix over a field.
define matrix2_inv[F: Field](n: Nat, a: Matrix[F, n.suc.suc, n.suc.suc]) -> Matrix[F, n.suc.suc, n.suc.suc] {
    Matrix[F, n.suc.suc, n.suc.suc].new(matrix2_inv_entry[F](n, a))
}
