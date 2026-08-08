/// The "Mul" typeclass represents anything that has a binary multiplication operator.
typeclass A: Mul {
    mul: (A, A) -> A
}
