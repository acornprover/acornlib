from nat import Nat, pow_zero
from semiring import Semiring
from algebra.add_monoid import is_add_monoid_hom, identity_fn_is_add_monoid_hom, compose_is_add_monoid_hom
from algebra.monoid.monoid import is_monoid_hom, identity_fn_is_monoid_hom, compose_is_monoid_hom
from data.basic.functions import identity_fn, compose, compose_assoc, compose_identity_left, compose_identity_right,
    function_eq_transport_predicate, function_eq_transport_predicate_rev
from data.basic.relation_transport import preserves_binary_op

/// True if a function between semirings preserves addition, multiplication, and both identities.
define is_semiring_hom[S: Semiring, T: Semiring](f: S -> T) -> Bool {
    is_add_monoid_hom(f) and is_monoid_hom(f)
}

/// A semiring homomorphism preserving addition, multiplication, zero, and one.
structure SemiringHom[S: Semiring, T: Semiring] {
    /// The mapping for the homomorphism.
    hom: S -> T
} constraint {
    is_semiring_hom(hom)
}

/// Semiring homomorphism extensionality: two homomorphisms are equal when they agree on every input.
theorem semiring_hom_ext[S: Semiring, T: Semiring](f: SemiringHom[S, T], g: SemiringHom[S, T]) {
    (forall(a: S) { f.hom(a) = g.hom(a) }) implies f = g
} by {
    f.hom = g.hom
}

attributes SemiringHom[S: Semiring, T: Semiring] {
    /// Semiring homomorphism extensionality from pointwise equality of the homomorphism.
    let ext = semiring_hom_ext[S, T]
}

/// Equal semiring homomorphisms have equal underlying functions.
theorem semiring_hom_eq_hom[S: Semiring, T: Semiring](f: SemiringHom[S, T], g: SemiringHom[S, T]) {
    f = g implies f.hom = g.hom
}

/// Equal semiring homomorphisms have equal values at every element.
theorem semiring_hom_eq_apply[S: Semiring, T: Semiring](f: SemiringHom[S, T], g: SemiringHom[S, T], a: S) {
    f = g implies f.hom(a) = g.hom(a)
} by {
    f.hom = g.hom
    f.hom(a) = g.hom(a)
}

/// Equality of semiring homomorphisms transports predicates on semiring homomorphisms.
theorem semiring_hom_eq_transport_predicate[S: Semiring, T: Semiring](
    p: SemiringHom[S, T] -> Bool,
    f: SemiringHom[S, T],
    g: SemiringHom[S, T]
) {
    f = g and p(f) implies p(g)
}

/// Equality of semiring homomorphisms transports predicates on semiring homomorphisms in the reverse direction.
theorem semiring_hom_eq_transport_predicate_rev[S: Semiring, T: Semiring](
    p: SemiringHom[S, T] -> Bool,
    f: SemiringHom[S, T],
    g: SemiringHom[S, T]
) {
    f = g and p(g) implies p(f)
}

/// Equality of underlying functions determines equality of semiring homomorphisms.
theorem semiring_hom_eq_of_hom_eq[S: Semiring, T: Semiring](f: SemiringHom[S, T], g: SemiringHom[S, T]) {
    f.hom = g.hom implies f = g
} by {
    forall(a: S) {
        f.hom(a) = g.hom(a)
    }
    semiring_hom_ext(f, g)
}

/// Pointwise equality determines equality of semiring homomorphisms.
theorem semiring_hom_eq_of_apply_eq[S: Semiring, T: Semiring](f: SemiringHom[S, T], g: SemiringHom[S, T]) {
    (forall(a: S) { f.hom(a) = g.hom(a) }) implies f = g
} by {
    semiring_hom_ext(f, g)
}

/// A semiring homomorphism preserves the additive identity.
theorem semiring_hom_zero[S: Semiring, T: Semiring](f: SemiringHom[S, T]) {
    f.hom(S.0) = T.0
} by {
    is_semiring_hom(f.hom)
    is_add_monoid_hom(f.hom) and is_monoid_hom(f.hom)
    is_add_monoid_hom(f.hom)
    is_add_monoid_hom(f.hom) = (f.hom(S.0) = T.0 and forall(x: S, y: S) {
        f.hom(x + y) = f.hom(x) + f.hom(y)
    })
    f.hom(S.0) = T.0 and forall(x: S, y: S) {
        f.hom(x + y) = f.hom(x) + f.hom(y)
    }
}

/// A semiring homomorphism preserves the multiplicative identity.
theorem semiring_hom_one[S: Semiring, T: Semiring](f: SemiringHom[S, T]) {
    f.hom(S.1) = T.1
} by {
    is_semiring_hom(f.hom)
    is_add_monoid_hom(f.hom) and is_monoid_hom(f.hom)
    is_monoid_hom(f.hom)
    is_monoid_hom(f.hom) = (f.hom(S.1) = T.1 and forall(x: S, y: S) {
        f.hom(x * y) = f.hom(x) * f.hom(y)
    })
    f.hom(S.1) = T.1 and forall(x: S, y: S) {
        f.hom(x * y) = f.hom(x) * f.hom(y)
    }
}

/// A semiring homomorphism preserves addition.
theorem semiring_hom_add[S: Semiring, T: Semiring](f: SemiringHom[S, T], a: S, b: S) {
    f.hom(a + b) = f.hom(a) + f.hom(b)
} by {
    is_semiring_hom(f.hom)
    is_add_monoid_hom(f.hom) and is_monoid_hom(f.hom)
    is_add_monoid_hom(f.hom)
    is_add_monoid_hom(f.hom) = (f.hom(S.0) = T.0 and forall(x: S, y: S) {
        f.hom(x + y) = f.hom(x) + f.hom(y)
    })
    f.hom(S.0) = T.0 and forall(x: S, y: S) {
        f.hom(x + y) = f.hom(x) + f.hom(y)
    }
    forall(x: S, y: S) {
        f.hom(x + y) = f.hom(x) + f.hom(y)
    }
    f.hom(a + b) = f.hom(a) + f.hom(b) and f.hom(a * b) = f.hom(a) * f.hom(b)
    f.hom(a + b) = f.hom(a) + f.hom(b)
}

/// A semiring homomorphism preserves multiplication.
theorem semiring_hom_mul[S: Semiring, T: Semiring](f: SemiringHom[S, T], a: S, b: S) {
    f.hom(a * b) = f.hom(a) * f.hom(b)
} by {
    is_semiring_hom(f.hom)
    is_add_monoid_hom(f.hom) and is_monoid_hom(f.hom)
    is_monoid_hom(f.hom)
    is_monoid_hom(f.hom) = (f.hom(S.1) = T.1 and forall(x: S, y: S) {
        f.hom(x * y) = f.hom(x) * f.hom(y)
    })
    f.hom(S.1) = T.1 and forall(x: S, y: S) {
        f.hom(x * y) = f.hom(x) * f.hom(y)
    }
    forall(x: S, y: S) {
        f.hom(x * y) = f.hom(x) * f.hom(y)
    }
    f.hom(a + b) = f.hom(a) + f.hom(b) and f.hom(a * b) = f.hom(a) * f.hom(b)
    f.hom(a * b) = f.hom(a) * f.hom(b)
}

/// A semiring homomorphism preserves powers.
theorem semiring_hom_pow[S: Semiring, T: Semiring](f: SemiringHom[S, T], a: S, n: Nat) {
    f.hom(a.pow(n)) = f.hom(a).pow(n)
} by {
    define p(k: Nat) -> Bool {
        f.hom(a.pow(k)) = f.hom(a).pow(k)
    }

    a.pow(Nat.0) = S.1
    f.hom(S.1) = T.1
    f.hom(a).pow(Nat.0) = T.1
    p(Nat.0)

    forall(k: Nat) {
        if p(k) {
            f.hom(a.pow(k)) = f.hom(a).pow(k)
            a.pow(k.suc) = a * a.pow(k)
            semiring_hom_mul(f, a, a.pow(k))
            f.hom(a * a.pow(k)) = f.hom(a) * f.hom(a.pow(k))
            f.hom(a) * f.hom(a.pow(k)) = f.hom(a) * f.hom(a).pow(k)
            f.hom(a) * f.hom(a).pow(k) = f.hom(a).pow(k.suc)
            p(k.suc)
        }
    }
}

/// Equality of functions transports the property of being a semiring homomorphism.
theorem function_eq_transport_semiring_hom[S: Semiring, T: Semiring](f: S -> T, g: S -> T) {
    f = g and is_semiring_hom(f) implies is_semiring_hom(g)
} by {
    function_eq_transport_predicate(is_semiring_hom[S, T], f, g)
}

/// Equality of functions transports the property of being a semiring homomorphism in the reverse direction.
theorem function_eq_transport_semiring_hom_rev[S: Semiring, T: Semiring](f: S -> T, g: S -> T) {
    f = g and is_semiring_hom(g) implies is_semiring_hom(f)
} by {
    function_eq_transport_predicate_rev(is_semiring_hom[S, T], f, g)
}

/// The identity function on a semiring preserves the semiring structure.
theorem identity_fn_is_semiring_hom[S: Semiring] {
    is_semiring_hom(identity_fn[S])
} by {
    identity_fn_is_add_monoid_hom[S]
    identity_fn_is_monoid_hom[S]
    is_add_monoid_hom(identity_fn[S]) and is_monoid_hom(identity_fn[S])
}

/// The composition of two semiring homomorphisms preserves the semiring structure.
theorem compose_is_semiring_hom[S: Semiring, T: Semiring, U: Semiring](f: T -> U, g: S -> T) {
    is_semiring_hom(f) and is_semiring_hom(g) implies is_semiring_hom(compose(f, g))
} by {
    is_add_monoid_hom(f) and is_monoid_hom(f)
    is_add_monoid_hom(g) and is_monoid_hom(g)
    is_add_monoid_hom(f)
    is_add_monoid_hom(g)
    is_monoid_hom(f)
    is_monoid_hom(g)
    compose_is_add_monoid_hom(f, g)
    compose_is_monoid_hom(f, g)
    is_add_monoid_hom(compose(f, g))
    is_monoid_hom(compose(f, g))
    is_add_monoid_hom(compose(f, g)) and is_monoid_hom(compose(f, g))
    is_semiring_hom(compose(f, g))
}

/// The identity homomorphism on a semiring.
let identity_semiring_hom[S: Semiring]: SemiringHom[S, S] satisfy {
    SemiringHom.new(identity_fn[S]) = Option.some(identity_semiring_hom)
}

/// The underlying function of the identity semiring homomorphism is the identity function.
theorem identity_semiring_hom_hom[S: Semiring] {
    identity_semiring_hom[S].hom = identity_fn[S]
}

/// The composition of two semiring homomorphisms.
let compose_semiring_hom[S: Semiring, T: Semiring, U: Semiring](f: SemiringHom[T, U], g: SemiringHom[S, T]) -> result: SemiringHom[S, U] satisfy {
    SemiringHom.new(compose(f.hom, g.hom)) = Option.some(result)
} by {
    is_semiring_hom(f.hom)
    is_semiring_hom(g.hom)
    compose_is_semiring_hom(f.hom, g.hom)
    is_semiring_hom(compose(f.hom, g.hom))
}

/// The underlying function of a composition of semiring homomorphisms is the composition of underlying functions.
theorem compose_semiring_hom_hom[S: Semiring, T: Semiring, U: Semiring](f: SemiringHom[T, U], g: SemiringHom[S, T]) {
    compose_semiring_hom(f, g).hom = compose(f.hom, g.hom)
}

/// Composition of semiring homomorphisms is associative.
theorem compose_semiring_hom_assoc[S: Semiring, T: Semiring, U: Semiring, V: Semiring](
    f: SemiringHom[U, V], g: SemiringHom[T, U], h: SemiringHom[S, T]) {
    compose_semiring_hom(compose_semiring_hom(f, g), h) = compose_semiring_hom(f, compose_semiring_hom(g, h))
} by {
    let lhs = compose_semiring_hom(compose_semiring_hom(f, g), h)
    let rhs = compose_semiring_hom(f, compose_semiring_hom(g, h))
    lhs.hom = compose(compose_semiring_hom(f, g).hom, h.hom)
    compose_semiring_hom(f, g).hom = compose(f.hom, g.hom)
    lhs.hom = compose(compose(f.hom, g.hom), h.hom)
    rhs.hom = compose(f.hom, compose_semiring_hom(g, h).hom)
    compose_semiring_hom(g, h).hom = compose(g.hom, h.hom)
    rhs.hom = compose(f.hom, compose(g.hom, h.hom))
    compose_assoc(f.hom, g.hom, h.hom)
    lhs.hom = rhs.hom
}

/// Composing the identity semiring homomorphism on the left leaves a homomorphism unchanged.
theorem compose_semiring_hom_identity_left[S: Semiring, T: Semiring](f: SemiringHom[S, T]) {
    compose_semiring_hom(identity_semiring_hom[T], f) = f
} by {
    let lhs = compose_semiring_hom(identity_semiring_hom[T], f)
    lhs.hom = compose(identity_semiring_hom[T].hom, f.hom)
    identity_semiring_hom[T].hom = identity_fn[T]
    lhs.hom = compose(identity_fn[T], f.hom)
    compose_identity_left(f.hom)
    lhs.hom = f.hom
}

/// Composing the identity semiring homomorphism on the right leaves a homomorphism unchanged.
theorem compose_semiring_hom_identity_right[S: Semiring, T: Semiring](f: SemiringHom[S, T]) {
    compose_semiring_hom(f, identity_semiring_hom[S]) = f
} by {
    let lhs = compose_semiring_hom(f, identity_semiring_hom[S])
    lhs.hom = compose(f.hom, identity_semiring_hom[S].hom)
    identity_semiring_hom[S].hom = identity_fn[S]
    lhs.hom = compose(f.hom, identity_fn[S])
    compose_identity_right(f.hom)
    lhs.hom = f.hom
}
