from algebra.comm_monoid import CommMonoid
from algebra.comm_semigroup import CommSemigroup
from algebra.monoid.monoid import Monoid, MonoidHom, monoid_hom_mul
from nat import Nat, pow_distrib_mul
from algebra.semigroup import Semigroup

/// True when an element of a multiplicative semigroup is a square.
define is_square[M: Semigroup](a: M) -> Bool {
    exists(r: M) { a = r * r }
}

/// The square predicate unfolds to the existence of a self-product.
theorem is_square_iff_exists_mul_self[M: Semigroup](a: M) {
    is_square[M](a) = exists(r: M) { a = r * r }
}

/// A self-product is a square.
theorem is_square_mul_self[M: Semigroup](r: M) {
    is_square[M](r * r)
} by {
    exists(s: M) { r * r = s * s }
}

/// The identity element of a monoid is a square.
theorem is_square_one[M: Monoid] {
    is_square[M](M.1)
} by {
    M.1 * M.1 = M.1
    exists(r: M) { M.1 = r * r }
}

/// A second power is a square.
theorem is_square_pow_two[M: Monoid](a: M) {
    is_square[M](a.pow(Nat.2))
} by {
    a.pow(Nat.2) = a * a
    is_square_mul_self[M](a)
    is_square[M](a * a)
    is_square[M](a.pow(Nat.2))
}

/// Powers of a square are squares in a commutative monoid.
theorem is_square_pow[M: CommMonoid](a: M, n: Nat) {
    is_square[M](a) implies is_square[M](a.pow(n))
} by {
    if is_square[M](a) {
        is_square[M](a) = exists(r: M) { a = r * r }
        let r: M satisfy { a = r * r }
        a = r * r
        a.pow(n) = (r * r).pow(n)
        pow_distrib_mul[M](r, r, n)
        (r * r).pow(n) = r.pow(n) * r.pow(n)
        a.pow(n) = r.pow(n) * r.pow(n)
        let w: M = r.pow(n)
        a.pow(n) = w * w
        exists(s: M) { a.pow(n) = s * s }
    }
}

/// The product of two squares is a square in a commutative semigroup.
theorem is_square_mul[M: CommSemigroup](a: M, b: M) {
    is_square[M](a) and is_square[M](b) implies is_square[M](a * b)
} by {
    if is_square[M](a) and is_square[M](b) {
        is_square[M](a) = exists(r: M) { a = r * r }
        is_square[M](b) = exists(s: M) { b = s * s }
        let r: M satisfy { a = r * r }
        let s: M satisfy { b = s * s }
        a = r * r
        b = s * s
        a * b = (r * r) * (s * s)
        (r * r) * (s * s) = (r * s) * (r * s)
        a * b = (r * s) * (r * s)
        let w: M = r * s
        a * b = w * w
        exists(t: M) { a * b = t * t }
    }
}


/// A monoid homomorphism carries squares to squares.
theorem is_square_map[M: Monoid, N: Monoid](f: MonoidHom[M, N], a: M) {
    is_square[M](a) implies is_square[N](f.hom(a))
} by {
    if is_square[M](a) {
        is_square[M](a) = exists(r: M) { a = r * r }
        let r: M satisfy { a = r * r }
        a = r * r
        f.hom(a) = f.hom(r * r)
        monoid_hom_mul(f, r, r)
        f.hom(r * r) = f.hom(r) * f.hom(r)
        f.hom(a) = f.hom(r) * f.hom(r)
        let w: N = f.hom(r)
        f.hom(a) = w * w
        exists(s: N) { f.hom(a) = s * s }
    }
}
