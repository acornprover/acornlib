from algebra.add_semigroup import AddSemigroup
from algebra.zero import Zero
from data.basic.functions import identity_fn, compose, compose_assoc, compose_identity_left, compose_identity_right,
    function_eq_transport_predicate, function_eq_transport_predicate_rev

/// An additive monoid is an additive semigroup with an identity element.
typeclass A: AddMonoid extends AddSemigroup, Zero {
    /// The identity element must satisfy the identity property on the right.
    add_identity_right(a: A) {
        a + A.0 = a
    }

    /// The identity element must satisfy the identity property on the left.
    add_identity_left(a: A) {
        A.0 + a = a
    }
}

/// True if a function between additive monoids preserves the operation and the identity.
define is_add_monoid_hom[A: AddMonoid, B: AddMonoid](f: A -> B) -> Bool {
    f(A.0) = B.0 and forall(a: A, b: A) {
        f(a + b) = f(a) + f(b)
    }
}

/// The trivial additive monoid homomorphism maps every element of A to the identity element of B.
let trivial_add_monoid_hom[A: AddMonoid, B: AddMonoid]: A -> B = function(a: A) {
    B.0
}

/// The trivial additive monoid homomorphism preserves the operation and the identity.
theorem trivial_add_monoid_hom_is_hom[A: AddMonoid, B: AddMonoid] {
    is_add_monoid_hom(trivial_add_monoid_hom[A, B])
}

/// An additive monoid homomorphism that preserves the operation and the identity.
structure AddMonoidHom[A: AddMonoid, B: AddMonoid] {
    /// The mapping for the homomorphism.
    hom: A -> B
} constraint {
    is_add_monoid_hom(hom)
}

/// Additive monoid homomorphism extensionality: two homomorphisms are equal when they agree on every input.
theorem add_monoid_hom_ext[A: AddMonoid, B: AddMonoid](f: AddMonoidHom[A, B], g: AddMonoidHom[A, B]) {
    (forall(a: A) { f.hom(a) = g.hom(a) }) implies f = g
} by {
    if forall(a: A) { f.hom(a) = g.hom(a) } {
        f.hom = g.hom
    }
}

attributes AddMonoidHom[A: AddMonoid, B: AddMonoid] {
    /// Additive monoid homomorphism extensionality from pointwise equality of the homomorphism.
    let ext = add_monoid_hom_ext[A, B]
}

/// Equal additive monoid homomorphisms have equal underlying functions.
theorem add_monoid_hom_eq_hom[A: AddMonoid, B: AddMonoid](f: AddMonoidHom[A, B], g: AddMonoidHom[A, B]) {
    f = g implies f.hom = g.hom
}

/// Equal additive monoid homomorphisms have equal values at every element.
theorem add_monoid_hom_eq_apply[A: AddMonoid, B: AddMonoid](f: AddMonoidHom[A, B], g: AddMonoidHom[A, B], a: A) {
    f = g implies f.hom(a) = g.hom(a)
} by {
    if f = g {
        f.hom(a) = g.hom(a)
    }
}

/// Equality of additive monoid homomorphisms transports predicates on additive monoid homomorphisms.
theorem add_monoid_hom_eq_transport_predicate[A: AddMonoid, B: AddMonoid](
    p: AddMonoidHom[A, B] -> Bool,
    f: AddMonoidHom[A, B],
    g: AddMonoidHom[A, B]
) {
    f = g and p(f) implies p(g)
}

/// Equality of additive monoid homomorphisms transports predicates on additive monoid homomorphisms in the reverse direction.
theorem add_monoid_hom_eq_transport_predicate_rev[A: AddMonoid, B: AddMonoid](
    p: AddMonoidHom[A, B] -> Bool,
    f: AddMonoidHom[A, B],
    g: AddMonoidHom[A, B]
) {
    f = g and p(g) implies p(f)
}

/// Equality of underlying functions determines equality of additive monoid homomorphisms.
theorem add_monoid_hom_eq_of_hom_eq[A: AddMonoid, B: AddMonoid](f: AddMonoidHom[A, B], g: AddMonoidHom[A, B]) {
    f.hom = g.hom implies f = g
} by {
    if f.hom = g.hom {
        forall(a: A) {
            f.hom(a) = g.hom(a)
        }
        add_monoid_hom_ext(f, g)
    }
}

/// Pointwise equality determines equality of additive monoid homomorphisms.
theorem add_monoid_hom_eq_of_apply_eq[A: AddMonoid, B: AddMonoid](f: AddMonoidHom[A, B], g: AddMonoidHom[A, B]) {
    (forall(a: A) { f.hom(a) = g.hom(a) }) implies f = g
} by {
    if forall(a: A) { f.hom(a) = g.hom(a) } {
        add_monoid_hom_ext(f, g)
    }
}

/// An additive monoid homomorphism preserves the identity element.
theorem add_monoid_hom_zero[A: AddMonoid, B: AddMonoid](f: AddMonoidHom[A, B]) {
    f.hom(A.0) = B.0
}

/// An additive monoid homomorphism preserves the operation.
theorem add_monoid_hom_add[A: AddMonoid, B: AddMonoid](f: AddMonoidHom[A, B], a: A, b: A) {
    f.hom(a + b) = f.hom(a) + f.hom(b)
} by {
    f.hom(A.0) = B.0 and forall(x: A, y: A) {
        f.hom(x + y) = f.hom(x) + f.hom(y)
    }
}

/// Equality of functions transports the property of being an additive monoid homomorphism.
theorem function_eq_transport_add_monoid_hom[A: AddMonoid, B: AddMonoid](f: A -> B, g: A -> B) {
    f = g and is_add_monoid_hom(f) implies is_add_monoid_hom(g)
} by {
    if f = g and is_add_monoid_hom(f) {
        function_eq_transport_predicate(is_add_monoid_hom[A, B], f, g)
    }
}

/// Equality of functions transports the property of being an additive monoid homomorphism in the reverse direction.
theorem function_eq_transport_add_monoid_hom_rev[A: AddMonoid, B: AddMonoid](f: A -> B, g: A -> B) {
    f = g and is_add_monoid_hom(g) implies is_add_monoid_hom(f)
} by {
    if f = g and is_add_monoid_hom(g) {
        function_eq_transport_predicate_rev(is_add_monoid_hom[A, B], f, g)
    }
}

/// The identity function on an additive monoid preserves the additive monoid structure.
theorem identity_fn_is_add_monoid_hom[A: AddMonoid] {
    is_add_monoid_hom(identity_fn[A])
} by {
    forall(a: A, b: A) {
        identity_fn[A](a + b) = identity_fn[A](a) + identity_fn[A](b)
    }
}

/// The composition of two additive monoid homomorphisms preserves the additive monoid structure.
theorem compose_is_add_monoid_hom[A: AddMonoid, B: AddMonoid, C: AddMonoid](f: B -> C, g: A -> B) {
    is_add_monoid_hom(f) and is_add_monoid_hom(g) implies is_add_monoid_hom(compose(f, g))
} by {
    if is_add_monoid_hom(f) and is_add_monoid_hom(g) {
        is_add_monoid_hom(f) = (f(B.0) = C.0 and forall(x: B, y: B) {
            f(x + y) = f(x) + f(y)
        })
        is_add_monoid_hom(g) = (g(A.0) = B.0 and forall(x: A, y: A) {
            g(x + y) = g(x) + g(y)
        })
        compose(f, g)(A.0) = C.0
        forall(a: A, b: A) {
            f(g(a) + g(b)) = f(g(a)) + f(g(b))
            compose(f, g)(a + b) = compose(f, g)(a) + compose(f, g)(b)
        }
        is_add_monoid_hom(compose(f, g))
    }
}

/// The identity homomorphism on an additive monoid.
let identity_add_monoid_hom[A: AddMonoid]: AddMonoidHom[A, A] satisfy {
    AddMonoidHom.new(identity_fn[A]) = Option.some(identity_add_monoid_hom)
}

/// The underlying function of the identity additive monoid homomorphism is the identity function.
theorem identity_add_monoid_hom_hom[A: AddMonoid] {
    identity_add_monoid_hom[A].hom = identity_fn[A]
}

/// The composition of two additive monoid homomorphisms.
let compose_add_monoid_hom[A: AddMonoid, B: AddMonoid, C: AddMonoid](f: AddMonoidHom[B, C], g: AddMonoidHom[A, B]) -> result: AddMonoidHom[A, C] satisfy {
    AddMonoidHom.new(compose(f.hom, g.hom)) = Option.some(result)
} by {
    compose_is_add_monoid_hom(f.hom, g.hom)
}

/// The underlying function of a composition of additive monoid homomorphisms is the composition of underlying functions.
theorem compose_add_monoid_hom_hom[A: AddMonoid, B: AddMonoid, C: AddMonoid](
    f: AddMonoidHom[B, C],
    g: AddMonoidHom[A, B]
) {
    compose_add_monoid_hom(f, g).hom = compose(f.hom, g.hom)
}

/// Composition of additive monoid homomorphisms is associative.
theorem compose_add_monoid_hom_assoc[A: AddMonoid, B: AddMonoid, C: AddMonoid, D: AddMonoid](
    f: AddMonoidHom[C, D],
    g: AddMonoidHom[B, C],
    h: AddMonoidHom[A, B]
) {
    compose_add_monoid_hom(compose_add_monoid_hom(f, g), h) = compose_add_monoid_hom(f, compose_add_monoid_hom(g, h))
} by {
    let lhs = compose_add_monoid_hom(compose_add_monoid_hom(f, g), h)
    let rhs = compose_add_monoid_hom(f, compose_add_monoid_hom(g, h))
    lhs.hom = compose(compose_add_monoid_hom(f, g).hom, h.hom)
    compose_add_monoid_hom(f, g).hom = compose(f.hom, g.hom)
    lhs.hom = compose(compose(f.hom, g.hom), h.hom)
    rhs.hom = compose(f.hom, compose_add_monoid_hom(g, h).hom)
    compose_add_monoid_hom(g, h).hom = compose(g.hom, h.hom)
    rhs.hom = compose(f.hom, compose(g.hom, h.hom))
    compose_assoc(f.hom, g.hom, h.hom)
    lhs.hom = rhs.hom
}

/// Composing the identity additive monoid homomorphism on the left leaves a homomorphism unchanged.
theorem compose_add_monoid_hom_identity_left[A: AddMonoid, B: AddMonoid](f: AddMonoidHom[A, B]) {
    compose_add_monoid_hom(identity_add_monoid_hom[B], f) = f
} by {
    let lhs = compose_add_monoid_hom(identity_add_monoid_hom[B], f)
    compose_identity_left(f.hom)
}

/// Composing the identity additive monoid homomorphism on the right leaves a homomorphism unchanged.
theorem compose_add_monoid_hom_identity_right[A: AddMonoid, B: AddMonoid](f: AddMonoidHom[A, B]) {
    compose_add_monoid_hom(f, identity_add_monoid_hom[A]) = f
} by {
    let lhs = compose_add_monoid_hom(f, identity_add_monoid_hom[A])
    compose_identity_right(f.hom)
}