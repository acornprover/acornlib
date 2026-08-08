/// Kernel-based injectivity criteria for algebraic homomorphisms: a
/// homomorphism is injective exactly when its kernel is trivial.

from algebra.group import Group, GroupHom, group_hom_mul, group_hom_inv, group_hom_one
from algebra.add_group import AddGroup, AddGroupHom, add_group_hom_add, add_group_hom_neg,
    add_group_hom_zero
from algebra.ring.ring import Ring
from algebra.ring.ring_hom import RingHom, ring_hom_add, ring_hom_neg, ring_hom_zero
from data.basic.functions import is_injective_fn
from algebra.field.field import Field
from algebra.field.field_hom import FieldHom, field_hom_mul, field_hom_one,
    field_hom_to_ring_hom, field_hom_to_ring_hom_hom

/// True if the only group element whose image is the identity is the identity.
define is_trivial_group_hom_kernel[G: Group, H: Group](f: GroupHom[G, H]) -> Bool {
    forall(a: G) {
        f.hom(a) = H.1 implies a = G.1
    }
}

/// Two elements share a group-homomorphism image exactly when the product of
/// the inverse of the first with the second lies in the kernel.
theorem group_hom_eq_iff_inv_mul_one[G: Group, H: Group](f: GroupHom[G, H], a: G, b: G) {
    (f.hom(a) = f.hom(b)) = (f.hom(a.inverse * b) = H.1)
} by {
    group_hom_mul(f, a.inverse, b)
    group_hom_inv(f, a)
    f.hom(a.inverse * b) = f.hom(a).inverse * f.hom(b)
    if f.hom(a) = f.hom(b) {
        f.hom(a).inverse * f.hom(a) = H.1
        f.hom(a.inverse * b) = H.1
    }
    if f.hom(a.inverse * b) = H.1 {
        f.hom(a).inverse * f.hom(b) = H.1
        f.hom(a) * (f.hom(a).inverse * f.hom(b)) = f.hom(a) * H.1
        f.hom(a) * (f.hom(a).inverse * f.hom(b)) = f.hom(b)
        f.hom(a) = f.hom(b)
    }
}

/// A group homomorphism is injective exactly when its kernel is trivial.
theorem group_hom_injective_iff_trivial_kernel[G: Group, H: Group](f: GroupHom[G, H]) {
    is_injective_fn(f.hom) = is_trivial_group_hom_kernel(f)
} by {
    is_injective_fn(f.hom) = forall(x: G, y: G) {
        f.hom(x) = f.hom(y) implies x = y
    }
    is_trivial_group_hom_kernel(f) = forall(a: G) {
        f.hom(a) = H.1 implies a = G.1
    }
    group_hom_one(f)
    if is_injective_fn(f.hom) {
        forall(a: G) {
            if f.hom(a) = H.1 {
                a = G.1
            }
        }
        is_trivial_group_hom_kernel(f)
    }
    if is_trivial_group_hom_kernel(f) {
        forall(x: G, y: G) {
            if f.hom(x) = f.hom(y) {
                group_hom_eq_iff_inv_mul_one(f, x, y)
                f.hom(x.inverse * y) = H.1
                x.inverse * y = G.1
                x * (x.inverse * y) = x * G.1
                x * (x.inverse * y) = y
                x = y
            }
        }
        is_injective_fn(f.hom)
    }
}

/// True if the only group element whose image is the zero is the zero.
define is_trivial_add_group_hom_kernel[A: AddGroup, B: AddGroup](f: AddGroupHom[A, B]) -> Bool {
    forall(a: A) {
        f.hom(a) = B.0 implies a = A.0
    }
}

/// Two elements share an additive-group-homomorphism image exactly when the
/// difference of the second and the first lies in the kernel.
theorem add_group_hom_eq_iff_sub_zero[A: AddGroup, B: AddGroup](f: AddGroupHom[A, B], a: A, b: A) {
    (f.hom(a) = f.hom(b)) = (f.hom(b - a) = B.0)
} by {
    b - a = b + -a
    add_group_hom_add(f, b, -a)
    add_group_hom_neg(f, a)
    f.hom(b - a) = f.hom(b) + -f.hom(a)
    if f.hom(a) = f.hom(b) {
        f.hom(b - a) = B.0
    }
    if f.hom(b - a) = B.0 {
        f.hom(b) + -f.hom(a) = B.0
        (f.hom(b) + -f.hom(a)) + f.hom(a) = B.0 + f.hom(a)
        (f.hom(b) + -f.hom(a)) + f.hom(a) = f.hom(b)
        f.hom(a) = f.hom(b)
    }
}

/// An additive group homomorphism is injective exactly when its kernel is trivial.
theorem add_group_hom_injective_iff_trivial_kernel[A: AddGroup, B: AddGroup](f: AddGroupHom[A, B]) {
    is_injective_fn(f.hom) = is_trivial_add_group_hom_kernel(f)
} by {
    is_injective_fn(f.hom) = forall(x: A, y: A) {
        f.hom(x) = f.hom(y) implies x = y
    }
    is_trivial_add_group_hom_kernel(f) = forall(a: A) {
        f.hom(a) = B.0 implies a = A.0
    }
    add_group_hom_zero(f)
    if is_injective_fn(f.hom) {
        forall(a: A) {
            if f.hom(a) = B.0 {
                a = A.0
            }
        }
        is_trivial_add_group_hom_kernel(f)
    }
    if is_trivial_add_group_hom_kernel(f) {
        forall(x: A, y: A) {
            if f.hom(x) = f.hom(y) {
                add_group_hom_eq_iff_sub_zero(f, x, y)
                f.hom(y - x) = B.0
                y - x = A.0
                (y - x) + x = A.0 + x
                (y - x) + x = y
                x = y
            }
        }
        is_injective_fn(f.hom)
    }
}

/// True if the only ring element whose image is the zero is the zero.
define is_trivial_ring_hom_kernel[R: Ring, S: Ring](f: RingHom[R, S]) -> Bool {
    forall(a: R) {
        f.hom(a) = S.0 implies a = R.0
    }
}

/// Two elements share a ring-homomorphism image exactly when the difference of
/// the second and the first lies in the kernel.
theorem ring_hom_eq_iff_sub_zero[R: Ring, S: Ring](f: RingHom[R, S], a: R, b: R) {
    (f.hom(a) = f.hom(b)) = (f.hom(b - a) = S.0)
} by {
    b - a = b + -a
    ring_hom_add(f, b, -a)
    ring_hom_neg(f, a)
    f.hom(b - a) = f.hom(b) + -f.hom(a)
    if f.hom(a) = f.hom(b) {
        f.hom(b) + -f.hom(b) = S.0
        f.hom(b - a) = S.0
    }
    if f.hom(b - a) = S.0 {
        f.hom(b) + -f.hom(a) = S.0
        (f.hom(b) + -f.hom(a)) + f.hom(a) = S.0 + f.hom(a)
        (f.hom(b) + -f.hom(a)) + f.hom(a) = f.hom(b)
        f.hom(a) = f.hom(b)
    }
}

/// A ring homomorphism is injective exactly when its kernel is trivial.
theorem ring_hom_injective_iff_trivial_kernel[R: Ring, S: Ring](f: RingHom[R, S]) {
    is_injective_fn(f.hom) = is_trivial_ring_hom_kernel(f)
} by {
    is_injective_fn(f.hom) = forall(x: R, y: R) {
        f.hom(x) = f.hom(y) implies x = y
    }
    is_trivial_ring_hom_kernel(f) = forall(a: R) {
        f.hom(a) = S.0 implies a = R.0
    }
    ring_hom_zero(f)
    if is_injective_fn(f.hom) {
        forall(a: R) {
            if f.hom(a) = S.0 {
                a = R.0
            }
        }
        is_trivial_ring_hom_kernel(f)
    }
    if is_trivial_ring_hom_kernel(f) {
        forall(x: R, y: R) {
            if f.hom(x) = f.hom(y) {
                ring_hom_eq_iff_sub_zero(f, x, y)
                f.hom(y - x) = S.0
                y - x = R.0
                (y - x) + x = R.0 + x
                (y - x) + x = y
                x = y
            }
        }
        is_injective_fn(f.hom)
    }
}

/// Two elements share a field-homomorphism image exactly when the difference
/// of the second and the first lies in the kernel.
theorem field_hom_eq_iff_sub_zero[F: Field, E: Field](f: FieldHom[F, E], a: F, b: F) {
    (f.hom(a) = f.hom(b)) = (f.hom(b - a) = E.0)
} by {
    field_hom_to_ring_hom_hom(f)
    ring_hom_eq_iff_sub_zero(field_hom_to_ring_hom(f), a, b)
}

/// True if the only field element whose image is the zero is the zero.
define is_trivial_field_hom_kernel[F: Field, E: Field](f: FieldHom[F, E]) -> Bool {
    forall(a: F) {
        f.hom(a) = E.0 implies a = F.0
    }
}

/// A field homomorphism is injective, since the only field element mapping to
/// zero is zero.
theorem field_hom_is_injective[F: Field, E: Field](f: FieldHom[F, E]) {
    is_injective_fn(f.hom)
} by {
    field_hom_to_ring_hom_hom(f)
    forall(a: F) {
        if field_hom_to_ring_hom(f).hom(a) = E.0 {
            if a != F.0 {
                a * a.inverse = F.1
                field_hom_mul(f, a, a.inverse)
                E.0 * f.hom(a.inverse) = E.0
                field_hom_one(f)
                E.0 != E.1
                false
            }
            a = F.0
        }
    }
    is_trivial_ring_hom_kernel(field_hom_to_ring_hom(f))
    ring_hom_injective_iff_trivial_kernel(field_hom_to_ring_hom(f))
}

/// Every field homomorphism has trivial kernel: only zero maps to zero.
theorem field_hom_has_trivial_kernel[F: Field, E: Field](f: FieldHom[F, E]) {
    is_trivial_field_hom_kernel(f)
} by {
    field_hom_to_ring_hom_hom(f)
    forall(a: F) {
        if f.hom(a) = E.0 {
            ring_hom_zero(field_hom_to_ring_hom(f))
            field_hom_is_injective(f)
            a = F.0
        }
    }
}

/// A field homomorphism is injective exactly when its kernel is trivial. The
/// kernel is always trivial, so both sides are always true.
theorem field_hom_injective_iff_trivial_kernel[F: Field, E: Field](f: FieldHom[F, E]) {
    is_injective_fn(f.hom) = is_trivial_field_hom_kernel(f)
} by {
    field_hom_is_injective(f)
    field_hom_has_trivial_kernel(f)
}
