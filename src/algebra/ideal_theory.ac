/// Ideal theory: the principal ideal, the ideal generated by two elements,
/// the well-definedness of the quotient-ring operations, the principal-ideal
/// property of the integers, and the ideal of a point of the affine plane.

from comm_ring import CommRing
from algebra.ring.ideal import Ideal, is_ideal, ideal_subset, is_ideal_add, is_ideal_absorb,
    ideal_add_constraint, ideal_absorb_constraint, principal_ideal, ideal_sum
from data.basic.quotient_algebra import ideal_quotient_rel,
    ideal_quotient_rel_respects_add, ideal_quotient_rel_respects_mul
from data.basic.relation_transport import respects_add, respects_mul, respects_binary_op,
    respects_add_eq_respects_binary_op, respects_mul_eq_respects_binary_op
from int import Int, gcd_div_left, gcd_div_right, spans, spans_gcd, spans_mul
from algebra.algebraic_geometry import PlanePoint, ideal_of_point, is_ideal_of_point,
    ideal_of_point_is_ideal

// ---------------------------------------------------------------------------
// (a) The principal ideal
// ---------------------------------------------------------------------------

/// The principal ideal generated by an element is closed under addition.
theorem principal_ideal_add_closed[R: CommRing](a: R, x: R, y: R) {
    principal_ideal(a, x) and principal_ideal(a, y) implies principal_ideal(a, x + y)
} by {
    if principal_ideal(a, x) and principal_ideal(a, y) {
        let r: R satisfy { x = r * a }
        let s: R satisfy { y = s * a }
        x + y = (r + s) * a
        principal_ideal(a, x + y)
    }
}

/// The principal ideal generated by an element absorbs multiplication by ring elements.
theorem principal_ideal_absorb_closed[R: CommRing](a: R, r: R, x: R) {
    principal_ideal(a, x) implies principal_ideal(a, r * x)
} by {
    if principal_ideal(a, x) {
        let s: R satisfy { x = s * a }
        r * x = (r * s) * a
        principal_ideal(a, r * x)
    }
}

/// The principal ideal (a) = {r * a : r in R} is an ideal: it contains zero,
/// is closed under addition, and absorbs multiplication by any ring element.
theorem principal_ideal_is_ideal_closed[R: CommRing](a: R) {
    is_ideal(principal_ideal[R](a))
} by {
    principal_ideal(a, R.0)

    forall(x: R, y: R) {
        if principal_ideal(a, x) and principal_ideal(a, y) {
            principal_ideal_add_closed(a, x, y)
        }
    }
    ideal_add_constraint(principal_ideal[R](a))

    forall(r: R, x: R) {
        if principal_ideal(a, x) {
            principal_ideal_absorb_closed(a, r, x)
        }
    }
    ideal_absorb_constraint(principal_ideal[R](a))
}

// ---------------------------------------------------------------------------
// (b) The ideal generated by two elements
// ---------------------------------------------------------------------------

/// Membership in the ideal generated by two elements: elements of the form
/// r * a + s * b with r and s in the ring.
define two_generated_ideal[R: CommRing](a: R, b: R, x: R) -> Bool {
    exists(r: R, s: R) {
        x = r * a + s * b
    }
}

/// Membership in the ideal generated by two elements yields a witnessing
/// linear combination.
theorem two_generated_ideal_witness[R: CommRing](a: R, b: R, x: R) {
    two_generated_ideal(a, b, x) implies exists(r: R, s: R) {
        x = r * a + s * b
    }
} by {
    if two_generated_ideal(a, b, x) {
        exists(r: R, s: R) {
            x = r * a + s * b
        }
    }
}

/// A linear combination of the two generators belongs to the generated ideal.
theorem two_generated_ideal_intro[R: CommRing](a: R, b: R, r: R, s: R) {
    two_generated_ideal(a, b, r * a + s * b)
} by {
    r * a + s * b = r * a + s * b
}

/// The ideal generated by two elements contains the first generator.
theorem two_generated_ideal_contains_left[R: CommRing](a: R, b: R) {
    two_generated_ideal(a, b, a)
} by {
    a = R.1 * a + R.0 * b
}

/// The ideal generated by two elements contains the second generator.
theorem two_generated_ideal_contains_right[R: CommRing](a: R, b: R) {
    two_generated_ideal(a, b, b)
} by {
    b = R.0 * a + R.1 * b
}

/// The ideal generated by two elements is closed under addition.
theorem two_generated_ideal_add_closed[R: CommRing](a: R, b: R, x: R, y: R) {
    two_generated_ideal(a, b, x) and two_generated_ideal(a, b, y)
        implies two_generated_ideal(a, b, x + y)
} by {
    if two_generated_ideal(a, b, x) and two_generated_ideal(a, b, y) {
        let (r: R, s: R) satisfy { x = r * a + s * b }
        let (r2: R, s2: R) satisfy { y = r2 * a + s2 * b }
        x + y = (r * a + s * b) + (r2 * a + s2 * b)
        (r * a + s * b) + (r2 * a + s2 * b) = (r * a + r2 * a) + (s * b + s2 * b)
        (r * a + r2 * a) + (s * b + s2 * b) = (r + r2) * a + (s + s2) * b
        x + y = (r + r2) * a + (s + s2) * b
        two_generated_ideal(a, b, x + y)
    }
}

/// The ideal generated by two elements absorbs multiplication by ring elements.
theorem two_generated_ideal_absorb_closed[R: CommRing](a: R, b: R, r: R, x: R) {
    two_generated_ideal(a, b, x) implies two_generated_ideal(a, b, r * x)
} by {
    if two_generated_ideal(a, b, x) {
        let (s: R, t: R) satisfy { x = s * a + t * b }
        r * x = r * (s * a + t * b)
        r * (s * a + t * b) = r * (s * a) + r * (t * b)
        r * (s * a) + r * (t * b) = (r * s) * a + (r * t) * b
        r * x = (r * s) * a + (r * t) * b
        two_generated_ideal(a, b, r * x)
    }
}

/// The ideal generated by two elements is an ideal.
theorem two_generated_ideal_is_ideal[R: CommRing](a: R, b: R) {
    is_ideal(two_generated_ideal[R](a, b))
} by {
    R.0 * a = R.0
    R.0 * b = R.0
    R.0 * a + R.0 * b = R.0
    two_generated_ideal(a, b, R.0)

    forall(x: R, y: R) {
        if two_generated_ideal(a, b, x) and two_generated_ideal(a, b, y) {
            two_generated_ideal_add_closed(a, b, x, y)
        }
    }
    ideal_add_constraint(two_generated_ideal[R](a, b))

    forall(r: R, x: R) {
        if two_generated_ideal(a, b, x) {
            two_generated_ideal_absorb_closed(a, b, r, x)
        }
    }
    ideal_absorb_constraint(two_generated_ideal[R](a, b))
}

/// Any ideal containing both generators contains the ideal generated by them.
theorem two_generated_ideal_subset_of_contains_generators[R: CommRing](
    a: R, b: R, contains: R -> Bool
) {
    is_ideal(contains) and contains(a) and contains(b)
        implies ideal_subset(two_generated_ideal[R](a, b), contains)
} by {
    if is_ideal(contains) and contains(a) and contains(b) {
        forall(x: R) {
            if two_generated_ideal(a, b, x) {
                let (r: R, s: R) satisfy { x = r * a + s * b }
                is_ideal_absorb(contains, r, a)
                is_ideal_absorb(contains, s, b)
                contains(r * a)
                contains(s * b)
                is_ideal_add(contains, r * a, s * b)
                contains(x)
            }
        }
    }
}

/// The ideal generated by two elements is the sum of the two principal ideals.
theorem two_generated_ideal_eq_sum_principal[R: CommRing](a: R, b: R) {
    two_generated_ideal[R](a, b) = ideal_sum[R](principal_ideal[R](a), principal_ideal[R](b))
} by {
    forall(x: R) {
        if two_generated_ideal(a, b, x) {
            let (r: R, s: R) satisfy { x = r * a + s * b }
            ideal_sum(principal_ideal[R](a), principal_ideal[R](b), x) =
                exists(u: R, v: R) {
                    principal_ideal(a, u) and principal_ideal(b, v) and x = u + v
                }
            principal_ideal(a, r * a)
            principal_ideal(b, s * b)
            x = r * a + s * b
            ideal_sum(principal_ideal[R](a), principal_ideal[R](b), x)
        }
        if ideal_sum(principal_ideal[R](a), principal_ideal[R](b), x) {
            ideal_sum(principal_ideal[R](a), principal_ideal[R](b), x) =
                exists(u: R, v: R) {
                    principal_ideal(a, u) and principal_ideal(b, v) and x = u + v
                }
            let (u: R, v: R) satisfy {
                principal_ideal(a, u) and principal_ideal(b, v) and x = u + v
            }
            let r: R satisfy { u = r * a }
            let r2: R satisfy { v = r2 * b }
            x = r * a + r2 * b
            two_generated_ideal(a, b, x)
        }
        two_generated_ideal(a, b, x) = ideal_sum[R](principal_ideal[R](a), principal_ideal[R](b))(x)
    }
}

// ---------------------------------------------------------------------------
// (c) The integers form a principal ideal domain
// ---------------------------------------------------------------------------

// In Z every ideal is principal: Z is a principal ideal domain.
//
// Proof sketch. Let I be an ideal of Z. If I is the zero ideal, then I = (0).
// Otherwise I contains a nonzero element, hence (by closure under negation)
// a positive one; take the least positive element d of I (well-ordering of
// the naturals). For any x in I the division algorithm writes x = q * d + r
// with 0 <= r < d, and r = x - q * d lies in I because I is closed under
// addition and absorbs multiplication by ring elements; by minimality of d
// we get r = 0, so x = q * d is a multiple of d and I = (d).
//
// The two-generator form, which is the content used in practice, is proved
// below: (a, b) = (gcd(a, b)) in Z, via Bezout's identity (spans_gcd).
//
// theorem integer_ideal_is_principal(contains: Int -> Bool) {
//     is_ideal(contains) implies exists(d: Int) { contains = principal_ideal[Int](d) }
// }

/// In Z, the ideal generated by two elements is the principal ideal generated
/// by their gcd.
theorem integer_two_generated_ideal_eq_principal_gcd(a: Int, b: Int) {
    two_generated_ideal[Int](a, b) = principal_ideal[Int](a.gcd(b))
} by {
    forall(x: Int) {
        if two_generated_ideal(a, b, x) {
            let (r: Int, s: Int) satisfy { x = r * a + s * b }
            gcd_div_left(a, b)
            gcd_div_right(a, b)
            let m: Int satisfy { m * a.gcd(b) = a }
            let n: Int satisfy { n * a.gcd(b) = b }
            a = m * a.gcd(b)
            b = n * a.gcd(b)
            r * a = r * (m * a.gcd(b))
            r * (m * a.gcd(b)) = (r * m) * a.gcd(b)
            s * b = (s * n) * a.gcd(b)
            x = (r * m) * a.gcd(b) + (s * n) * a.gcd(b)
            (r * m) * a.gcd(b) + (s * n) * a.gcd(b) = (r * m + s * n) * a.gcd(b)
            x = (r * m + s * n) * a.gcd(b)
            principal_ideal(a.gcd(b), x)
        }
        if principal_ideal(a.gcd(b), x) {
            let r: Int satisfy { x = r * a.gcd(b) }
            spans_gcd(a, b)
            spans_mul(a, b, a.gcd(b), r)
            spans(a, b, r * a.gcd(b))
            r * a.gcd(b) = x
            spans(a, b, x)
            let (d: Int, e: Int) satisfy { d * a + e * b = x }
            two_generated_ideal(a, b, x)
        }
        two_generated_ideal(a, b, x) = principal_ideal(a.gcd(b), x)
    }
}

// ---------------------------------------------------------------------------
// (d) The quotient ring R/I
// ---------------------------------------------------------------------------

/// Addition on the quotient ring R/I is well-defined: congruent representatives
/// give congruent sums.
theorem ideal_quotient_add_well_defined[R: CommRing](
    i: Ideal[R], a1: R, b1: R, a2: R, b2: R
) {
    ideal_quotient_rel(i, a1, b1) and ideal_quotient_rel(i, a2, b2)
        implies ideal_quotient_rel(i, a1 + a2, b1 + b2)
} by {
    if ideal_quotient_rel(i, a1, b1) and ideal_quotient_rel(i, a2, b2) {
        ideal_quotient_rel_respects_add(i)
        respects_add_eq_respects_binary_op(ideal_quotient_rel(i))
        respects_add(ideal_quotient_rel(i)) = respects_binary_op(ideal_quotient_rel(i), R.add)
        respects_binary_op(ideal_quotient_rel(i), R.add)
        forall(x1: R, y1: R, x2: R, y2: R) {
            if ideal_quotient_rel(i, x1, y1) and ideal_quotient_rel(i, x2, y2) {
                ideal_quotient_rel(i, x1 + x2, y1 + y2)
            }
        }
        ideal_quotient_rel(i, a1 + a2, b1 + b2)
    }
}

/// Multiplication on the quotient ring R/I is well-defined: congruent
/// representatives give congruent products.
theorem ideal_quotient_mul_well_defined[R: CommRing](
    i: Ideal[R], a1: R, b1: R, a2: R, b2: R
) {
    ideal_quotient_rel(i, a1, b1) and ideal_quotient_rel(i, a2, b2)
        implies ideal_quotient_rel(i, a1 * a2, b1 * b2)
} by {
    if ideal_quotient_rel(i, a1, b1) and ideal_quotient_rel(i, a2, b2) {
        ideal_quotient_rel_respects_mul(i)
        respects_mul_eq_respects_binary_op(ideal_quotient_rel(i))
        respects_mul(ideal_quotient_rel(i)) = respects_binary_op(ideal_quotient_rel(i), R.mul)
        respects_binary_op(ideal_quotient_rel(i), R.mul)
        forall(x1: R, y1: R, x2: R, y2: R) {
            if ideal_quotient_rel(i, x1, y1) and ideal_quotient_rel(i, x2, y2) {
                ideal_quotient_rel(i, x1 * x2, y1 * y2)
            }
        }
        ideal_quotient_rel(i, a1 * a2, b1 * b2)
    }
}

// ---------------------------------------------------------------------------
// (e) The ideal of a point in the polynomial ring
// ---------------------------------------------------------------------------

/// The ideal of a point of the affine plane over R: the bivariate polynomials
/// vanishing at the point. This is an ideal of the bivariate polynomial ring;
/// the full verification (containing the zero polynomial, closure under
/// addition, and absorption of multiplication) is carried out in
/// `algebra/algebraic_geometry.ac`.
theorem point_ideal_is_ideal[R: CommRing](p: PlanePoint[R]) {
    is_ideal_of_point(p, ideal_of_point(p))
} by {
    ideal_of_point_is_ideal(p)
}
