/// Affine varieties in the plane: zero sets of bivariate polynomials, the
/// conic sections (parabola and unit circle), the ideal of a point, the
/// variety of an ideal generated by one polynomial, and a statement of the
/// Nullstellensatz.
///
/// Bivariate polynomials are built from the univariate polynomial API: a
/// bivariate polynomial over `R` is a polynomial in `Y` whose coefficients
/// are polynomials in `X`.  The inner ring `PolyC[R]` is a thin wrapper that
/// carries the ring typeclass instances (the polynomial library itself has
/// only additive instances), so that evaluation at a point can be expressed
/// as two univariate evaluations.

from nat import Nat, lt_suc
from semiring import Semiring
from comm_ring import CommRing
from algebra.field.field import Field, field_mul_eq_zero, field_mul_eq_zero_of_factor
from algebra.add import Add
from algebra.zero import Zero
from algebra.neg import Neg
from algebra.mul import Mul
from algebra.one import One
from algebra.add_semigroup import AddSemigroup
from algebra.add_comm_semigroup import AddCommSemigroup
from algebra.add_monoid import AddMonoid
from algebra.add_comm_monoid import AddCommMonoid
from algebra.add_group import AddGroup, inverse_left
from algebra.add_comm_group import AddCommGroup
from algebra.semigroup import Semigroup
from algebra.comm_semigroup import CommSemigroup
from algebra.monoid.monoid import Monoid
from algebra.comm_monoid import CommMonoid
from algebra.ring.ring import Ring, mul_zero_right, mul_zero_left
from polynomial import Polynomial, polynomial_constant, polynomial_monomial,
    polynomial_zero, polynomial_one, polynomial_add, polynomial_neg,
    polynomial_add_assoc, polynomial_add_comm, polynomial_add_zero_right,
    polynomial_add_zero_left, polynomial_add_neg_right, polynomial_ext_pointwise,
    polynomial_zero_coeff, polynomial_mul, polynomial_mul_coeff,
    polynomial_mul_coeff_apply, polynomial_mul_term_coeff,
    polynomial_eval, polynomial_eval_add, polynomial_eval_neg, polynomial_eval_sub,
    polynomial_eval_constant, polynomial_eval_one, polynomial_eval_zero,
    polynomial_eval_mul, coeff_eval, coeff_tail, polynomial_eval_bound,
    polynomial_eval_bound_eq_coeff_eval, polynomial_support_bounded_by,
    polynomial_eval_eq_eval_bound_of_support_bounded, coeff_zero_from,
    coeff_zero_from_at, coeff_zero_at, polynomial_monomial_coeff_of_ne,
    polynomial_monomial_coeff_self
from polynomial_mul_assoc import polynomial_mul_assoc
from polynomial_mul_comm import polynomial_mul_comm, polynomial_mul_one_right, polynomial_mul_one_left
from polynomial_mul_distrib import polynomial_mul_add_left
from data.basic.functions import function_extensionality
from data.nat.nat_range_sum import range_sum, range_sum_zero_fn, zero_fn
from data.nat.nat_range_sum_bridge import range_sum_eq_partial
from list import partial
from data.basic.set import Set, set_ext, union_contains_eq

numerals Nat

// ---------------------------------------------------------------------------
// The ring structure of univariate polynomials over a commutative ring
// ---------------------------------------------------------------------------

/// A univariate polynomial with commutative-ring coefficients, as a ring.
///
/// The library's `Polynomial` type carries no ring typeclass instances, so
/// this thin wrapper carries the additive and multiplicative instances.  All
/// laws are inherited from the polynomial theorems already in the library.
structure PolyC[R: CommRing] {
    /// The underlying polynomial.
    underlying: Polynomial[R]
}

/// Addition of wrapped polynomials is coefficientwise addition.
define polyc_add[R: CommRing](p: PolyC[R], q: PolyC[R]) -> PolyC[R] {
    PolyC[R].new(polynomial_add(p.underlying, q.underlying))
}

/// Multiplication of wrapped polynomials is convolution.
define polyc_mul[R: CommRing](p: PolyC[R], q: PolyC[R]) -> PolyC[R] {
    PolyC[R].new(polynomial_mul(p.underlying, q.underlying))
}

/// Negation of a wrapped polynomial is coefficientwise negation.
define polyc_neg[R: CommRing](p: PolyC[R]) -> PolyC[R] {
    PolyC[R].new(polynomial_neg(p.underlying))
}

/// The wrapped zero polynomial.
let polyc_zero[R: CommRing]: PolyC[R] = PolyC[R].new(polynomial_zero[R])

/// The wrapped one polynomial.
let polyc_one[R: CommRing]: PolyC[R] = PolyC[R].new(polynomial_one[R])

/// The constant polynomial with the given value, wrapped.
define polyc_constant[R: CommRing](r: R) -> PolyC[R] {
    PolyC[R].new(polynomial_constant(r))
}

/// The wrapped polynomial `X` (the monomial of degree one).
let polyc_x[R: CommRing]: PolyC[R] = PolyC[R].new(polynomial_monomial(Nat.1, R.1))

attributes PolyC[R: CommRing] {
    /// Addition of wrapped polynomials.
    define add(self, other: PolyC[R]) -> PolyC[R] {
        polyc_add(self, other)
    }

    /// Multiplication of wrapped polynomials.
    define mul(self, other: PolyC[R]) -> PolyC[R] {
        polyc_mul(self, other)
    }

    /// Negation of a wrapped polynomial.
    define neg(self) -> PolyC[R] {
        polyc_neg(self)
    }

    /// The wrapped zero polynomial.
    let zero: PolyC[R] = polyc_zero[R]

    /// The wrapped one polynomial.
    let one: PolyC[R] = polyc_one[R]
}

/// Wrapped polynomials have coefficientwise addition.
instance PolyC[R: CommRing]: Add {
    let add = PolyC[R].add
}

/// Wrapped polynomials have convolution multiplication.
instance PolyC[R: CommRing]: Mul {
    let mul = PolyC[R].mul
}

/// Wrapped polynomials have coefficientwise negation.
instance PolyC[R: CommRing]: Neg {
    let neg = PolyC[R].neg
}

/// Wrapped polynomials have the wrapped zero polynomial.
instance PolyC[R: CommRing]: Zero {
    let 0 = PolyC[R].zero
}

/// Wrapped polynomials have the wrapped one polynomial.
instance PolyC[R: CommRing]: One {
    let 1 = PolyC[R].one
}

/// The typeclass addition on wrapped polynomials is coefficientwise addition.
theorem polyc_typeclass_add_eq_polyc_add[R: CommRing](p: PolyC[R], q: PolyC[R]) {
    p + q = polyc_add(p, q)
} by {
    p + q = polyc_add(p, q)
}

/// The typeclass multiplication on wrapped polynomials is convolution.
theorem polyc_typeclass_mul_eq_polyc_mul[R: CommRing](p: PolyC[R], q: PolyC[R]) {
    p * q = polyc_mul(p, q)
} by {
    p * q = polyc_mul(p, q)
}

/// The typeclass negation of a wrapped polynomial is coefficientwise negation.
theorem polyc_typeclass_neg_eq_polyc_neg[R: CommRing](p: PolyC[R]) {
    -p = polyc_neg(p)
} by {
    -p = polyc_neg(p)
}

/// The typeclass zero of wrapped polynomials is the wrapped zero polynomial.
theorem polyc_typeclass_zero_eq_polyc_zero[R: CommRing] {
    Zero.0[PolyC[R]] = polyc_zero[R]
} by {
    Zero.0[PolyC[R]] = polyc_zero[R]
}

/// The typeclass one of wrapped polynomials is the wrapped one polynomial.
theorem polyc_typeclass_one_eq_polyc_one[R: CommRing] {
    One.1[PolyC[R]] = polyc_one[R]
} by {
    One.1[PolyC[R]] = polyc_one[R]
}

// -- the polynomial ring laws that the library states for polynomials ------

/// Multiplying a polynomial by the zero polynomial on the right gives zero.
theorem polynomial_mul_zero_right[R: CommRing](p: Polynomial[R]) {
    polynomial_mul(p, polynomial_zero[R]) = polynomial_zero[R]
} by {
    polynomial_ext_pointwise(polynomial_mul(p, polynomial_zero[R]), polynomial_zero[R])
    forall(k: Nat) {
        polynomial_mul_coeff_apply(p, polynomial_zero[R], k)
        polynomial_mul(p, polynomial_zero[R]).coeff(k) = polynomial_mul_coeff(p, polynomial_zero[R], k)
        polynomial_mul_coeff(p, polynomial_zero[R], k) =
            partial(function(i: Nat) { polynomial_mul_term_coeff(p, polynomial_zero[R], k, i) }, k.suc)
        forall(i: Nat) {
            function(j: Nat) { polynomial_mul_term_coeff(p, polynomial_zero[R], k, j) }(i) =
                polynomial_mul_term_coeff(p, polynomial_zero[R], k)(i)
        }
        function_extensionality(
            function(j: Nat) { polynomial_mul_term_coeff(p, polynomial_zero[R], k, j) },
            polynomial_mul_term_coeff(p, polynomial_zero[R], k))
        function(j: Nat) { polynomial_mul_term_coeff(p, polynomial_zero[R], k, j) } =
            polynomial_mul_term_coeff(p, polynomial_zero[R], k)
        polynomial_mul_coeff(p, polynomial_zero[R], k) =
            partial(polynomial_mul_term_coeff(p, polynomial_zero[R], k), k.suc)
        range_sum_eq_partial(polynomial_mul_term_coeff(p, polynomial_zero[R], k), k.suc)
        range_sum(polynomial_mul_term_coeff(p, polynomial_zero[R], k), k.suc) =
            partial(polynomial_mul_term_coeff(p, polynomial_zero[R], k), k.suc)
        partial(polynomial_mul_term_coeff(p, polynomial_zero[R], k), k.suc) =
            range_sum(polynomial_mul_term_coeff(p, polynomial_zero[R], k), k.suc)
        forall(i: Nat) {
            polynomial_mul_term_coeff(p, polynomial_zero[R], k, i) = p.coeff(i) * polynomial_zero[R].coeff(k - i)
            polynomial_zero_coeff[R](k - i)
            polynomial_zero[R].coeff(k - i) = R.0
            polynomial_mul_term_coeff(p, polynomial_zero[R], k, i) = p.coeff(i) * R.0
            mul_zero_right[R](p.coeff(i))
            p.coeff(i) * R.0 = R.0
            polynomial_mul_term_coeff(p, polynomial_zero[R], k, i) = R.0
            zero_fn[R](i) = R.0
            polynomial_mul_term_coeff(p, polynomial_zero[R], k, i) = zero_fn[R](i)
        }
        function_extensionality(polynomial_mul_term_coeff(p, polynomial_zero[R], k), zero_fn[R])
        polynomial_mul_term_coeff(p, polynomial_zero[R], k) = zero_fn[R]
        partial(polynomial_mul_term_coeff(p, polynomial_zero[R], k), k.suc) = partial(zero_fn[R], k.suc)
        range_sum_eq_partial(zero_fn[R], k.suc)
        range_sum(zero_fn[R], k.suc) = partial(zero_fn[R], k.suc)
        partial(zero_fn[R], k.suc) = range_sum(zero_fn[R], k.suc)
        range_sum_zero_fn[R](k.suc)
        range_sum(zero_fn[R], k.suc) = R.0
        partial(zero_fn[R], k.suc) = R.0
        partial(polynomial_mul_term_coeff(p, polynomial_zero[R], k), k.suc) = R.0
        polynomial_mul_coeff(p, polynomial_zero[R], k) = R.0
        polynomial_mul(p, polynomial_zero[R]).coeff(k) = R.0
        polynomial_zero_coeff[R](k)
        polynomial_zero[R].coeff(k) = R.0
        polynomial_mul(p, polynomial_zero[R]).coeff(k) = polynomial_zero[R].coeff(k)
    }
    polynomial_mul(p, polynomial_zero[R]) = polynomial_zero[R]
}

/// Multiplying a polynomial by the zero polynomial on the left gives zero.
theorem polynomial_mul_zero_left[R: CommRing](p: Polynomial[R]) {
    polynomial_mul(polynomial_zero[R], p) = polynomial_zero[R]
} by {
    polynomial_mul_comm(polynomial_zero[R], p)
    polynomial_mul(polynomial_zero[R], p) = polynomial_mul(p, polynomial_zero[R])
    polynomial_mul_zero_right(p)
}

/// Polynomial multiplication distributes over addition on the right.
theorem polynomial_mul_add_right[R: CommRing](p: Polynomial[R], q: Polynomial[R], r: Polynomial[R]) {
    polynomial_mul(polynomial_add(p, q), r) =
        polynomial_add(polynomial_mul(p, r), polynomial_mul(q, r))
} by {
    polynomial_mul_comm(polynomial_add(p, q), r)
    polynomial_mul(polynomial_add(p, q), r) = polynomial_mul(r, polynomial_add(p, q))
    polynomial_mul_add_left(r, p, q)
    polynomial_mul(r, polynomial_add(p, q)) =
        polynomial_add(polynomial_mul(r, p), polynomial_mul(r, q))
    polynomial_mul_comm(r, p)
    polynomial_mul(r, p) = polynomial_mul(p, r)
    polynomial_mul_comm(r, q)
    polynomial_mul(r, q) = polynomial_mul(q, r)
    polynomial_mul(polynomial_add(p, q), r) =
        polynomial_add(polynomial_mul(p, r), polynomial_mul(q, r))
}

// -- the additive typeclass laws for wrapped polynomials --------------------

/// Wrapped polynomial addition is associative.
theorem polyc_add_semigroup_law[R: CommRing](p: PolyC[R], q: PolyC[R], r: PolyC[R]) {
    p + (q + r) = (p + q) + r
} by {
    polyc_typeclass_add_eq_polyc_add(q, r)
    q + r = polyc_add(q, r)
    polyc_typeclass_add_eq_polyc_add(p, q + r)
    p + (q + r) = polyc_add(p, q + r)
    p + (q + r) = polyc_add(p, polyc_add(q, r))
    polyc_typeclass_add_eq_polyc_add(p, q)
    p + q = polyc_add(p, q)
    polyc_typeclass_add_eq_polyc_add(p + q, r)
    (p + q) + r = polyc_add(p + q, r)
    (p + q) + r = polyc_add(polyc_add(p, q), r)
    polyc_add(polyc_add(p, q), r) =
        PolyC[R].new(polynomial_add(polynomial_add(p.underlying, q.underlying), r.underlying))
    polyc_add(p, polyc_add(q, r)) =
        PolyC[R].new(polynomial_add(p.underlying, polynomial_add(q.underlying, r.underlying)))
    polynomial_add_assoc(p.underlying, q.underlying, r.underlying)
}

/// Wrapped polynomial addition is commutative.
theorem polyc_add_comm_semigroup_law[R: CommRing](p: PolyC[R], q: PolyC[R]) {
    p + q = q + p
} by {
    polyc_typeclass_add_eq_polyc_add(p, q)
    p + q = polyc_add(p, q)
    polyc_typeclass_add_eq_polyc_add(q, p)
    q + p = polyc_add(q, p)
    polyc_add(p, q) = PolyC[R].new(polynomial_add(p.underlying, q.underlying))
    polyc_add(q, p) = PolyC[R].new(polynomial_add(q.underlying, p.underlying))
    polynomial_add_comm(p.underlying, q.underlying)
}

/// The wrapped zero polynomial is a right identity for addition.
theorem polyc_add_monoid_right_law[R: CommRing](p: PolyC[R]) {
    p + Zero.0[PolyC[R]] = p
} by {
    polyc_typeclass_zero_eq_polyc_zero[R]
    Zero.0[PolyC[R]] = polyc_zero[R]
    polyc_typeclass_add_eq_polyc_add(p, Zero.0[PolyC[R]])
    p + Zero.0[PolyC[R]] = polyc_add(p, Zero.0[PolyC[R]])
    p + Zero.0[PolyC[R]] = polyc_add(p, polyc_zero[R])
    polyc_add(p, polyc_zero[R]) = PolyC[R].new(polynomial_add(p.underlying, polynomial_zero[R]))
    polynomial_add_zero_right(p.underlying)
    polyc_add(p, polyc_zero[R]) = PolyC[R].new(p.underlying)
    p = PolyC[R].new(p.underlying)
}

/// The wrapped zero polynomial is a left identity for addition.
theorem polyc_add_monoid_left_law[R: CommRing](p: PolyC[R]) {
    Zero.0[PolyC[R]] + p = p
} by {
    polyc_typeclass_zero_eq_polyc_zero[R]
    Zero.0[PolyC[R]] = polyc_zero[R]
    polyc_typeclass_add_eq_polyc_add(Zero.0[PolyC[R]], p)
    Zero.0[PolyC[R]] + p = polyc_add(Zero.0[PolyC[R]], p)
    Zero.0[PolyC[R]] + p = polyc_add(polyc_zero[R], p)
    polyc_add(polyc_zero[R], p) = PolyC[R].new(polynomial_add(polynomial_zero[R], p.underlying))
    polynomial_add_zero_left(p.underlying)
    polyc_add(polyc_zero[R], p) = PolyC[R].new(p.underlying)
    p = PolyC[R].new(p.underlying)
}

/// Adding the wrapped negative cancels.
theorem polyc_add_group_inverse_law[R: CommRing](p: PolyC[R]) {
    p + -p = Zero.0[PolyC[R]]
} by {
    polyc_typeclass_neg_eq_polyc_neg(p)
    -p = polyc_neg(p)
    polyc_typeclass_add_eq_polyc_add(p, -p)
    p + -p = polyc_add(p, -p)
    p + -p = polyc_add(p, polyc_neg(p))
    polyc_add(p, polyc_neg(p)) =
        PolyC[R].new(polynomial_add(p.underlying, polynomial_neg(p.underlying)))
    polynomial_neg(p.underlying) = p.underlying.neg
    polyc_add(p, polyc_neg(p)) =
        PolyC[R].new(polynomial_add(p.underlying, p.underlying.neg))
    polynomial_add_neg_right(p.underlying)
    polyc_add(p, polyc_neg(p)) = PolyC[R].new(polynomial_zero[R])
    polyc_typeclass_zero_eq_polyc_zero[R]
    Zero.0[PolyC[R]] = polyc_zero[R]
    polyc_zero[R] = PolyC[R].new(polynomial_zero[R])
    Zero.0[PolyC[R]] = PolyC[R].new(polynomial_zero[R])
}

// -- the multiplicative typeclass laws for wrapped polynomials ---------------

/// Wrapped polynomial multiplication is associative.
theorem polyc_mul_semigroup_law[R: CommRing](p: PolyC[R], q: PolyC[R], r: PolyC[R]) {
    p * (q * r) = (p * q) * r
} by {
    polyc_typeclass_mul_eq_polyc_mul(q, r)
    q * r = polyc_mul(q, r)
    polyc_typeclass_mul_eq_polyc_mul(p, q * r)
    p * (q * r) = polyc_mul(p, q * r)
    p * (q * r) = polyc_mul(p, polyc_mul(q, r))
    polyc_typeclass_mul_eq_polyc_mul(p, q)
    p * q = polyc_mul(p, q)
    polyc_typeclass_mul_eq_polyc_mul(p * q, r)
    (p * q) * r = polyc_mul(p * q, r)
    (p * q) * r = polyc_mul(polyc_mul(p, q), r)
    polyc_mul(polyc_mul(p, q), r) =
        PolyC[R].new(polynomial_mul(polynomial_mul(p.underlying, q.underlying), r.underlying))
    polyc_mul(p, polyc_mul(q, r)) =
        PolyC[R].new(polynomial_mul(p.underlying, polynomial_mul(q.underlying, r.underlying)))
    polynomial_mul_assoc(p.underlying, q.underlying, r.underlying)
}

/// Wrapped polynomial multiplication is commutative.
theorem polyc_mul_comm_semigroup_law[R: CommRing](p: PolyC[R], q: PolyC[R]) {
    p * q = q * p
} by {
    polyc_typeclass_mul_eq_polyc_mul(p, q)
    p * q = polyc_mul(p, q)
    polyc_typeclass_mul_eq_polyc_mul(q, p)
    q * p = polyc_mul(q, p)
    polyc_mul(p, q) = PolyC[R].new(polynomial_mul(p.underlying, q.underlying))
    polyc_mul(q, p) = PolyC[R].new(polynomial_mul(q.underlying, p.underlying))
    polynomial_mul_comm(p.underlying, q.underlying)
}

/// The wrapped one polynomial is a right identity for multiplication.
theorem polyc_monoid_right_law[R: CommRing](p: PolyC[R]) {
    p * One.1[PolyC[R]] = p
} by {
    polyc_typeclass_one_eq_polyc_one[R]
    One.1[PolyC[R]] = polyc_one[R]
    polyc_typeclass_mul_eq_polyc_mul(p, One.1[PolyC[R]])
    p * One.1[PolyC[R]] = polyc_mul(p, One.1[PolyC[R]])
    p * One.1[PolyC[R]] = polyc_mul(p, polyc_one[R])
    polyc_mul(p, polyc_one[R]) = PolyC[R].new(polynomial_mul(p.underlying, polynomial_one[R]))
    polynomial_mul_one_right(p.underlying)
    polyc_mul(p, polyc_one[R]) = PolyC[R].new(p.underlying)
    p = PolyC[R].new(p.underlying)
}

/// The wrapped one polynomial is a left identity for multiplication.
theorem polyc_monoid_left_law[R: CommRing](p: PolyC[R]) {
    One.1[PolyC[R]] * p = p
} by {
    polyc_typeclass_one_eq_polyc_one[R]
    One.1[PolyC[R]] = polyc_one[R]
    polyc_typeclass_mul_eq_polyc_mul(One.1[PolyC[R]], p)
    One.1[PolyC[R]] * p = polyc_mul(One.1[PolyC[R]], p)
    One.1[PolyC[R]] * p = polyc_mul(polyc_one[R], p)
    polyc_mul(polyc_one[R], p) = PolyC[R].new(polynomial_mul(polynomial_one[R], p.underlying))
    polynomial_mul_one_left(p.underlying)
    polyc_mul(polyc_one[R], p) = PolyC[R].new(p.underlying)
    p = PolyC[R].new(p.underlying)
}

// -- distributivity and zero absorption --------------------------------------

/// Wrapped polynomial multiplication distributes over addition on the left.
theorem polyc_semiring_distrib_left_law[R: CommRing](p: PolyC[R], q: PolyC[R], r: PolyC[R]) {
    p * (q + r) = p * q + p * r
} by {
    polyc_typeclass_add_eq_polyc_add(q, r)
    q + r = polyc_add(q, r)
    polyc_typeclass_mul_eq_polyc_mul(p, q + r)
    p * (q + r) = polyc_mul(p, q + r)
    p * (q + r) = polyc_mul(p, polyc_add(q, r))
    polyc_typeclass_add_eq_polyc_add(p * q, p * r)
    p * q + p * r = polyc_add(p * q, p * r)
    polyc_typeclass_mul_eq_polyc_mul(p, q)
    p * q = polyc_mul(p, q)
    polyc_typeclass_mul_eq_polyc_mul(p, r)
    p * r = polyc_mul(p, r)
    polyc_mul(p, polyc_add(q, r)) =
        PolyC[R].new(polynomial_mul(p.underlying, polynomial_add(q.underlying, r.underlying)))
    polyc_add(polyc_mul(p, q), polyc_mul(p, r)) =
        PolyC[R].new(polynomial_add(polynomial_mul(p.underlying, q.underlying),
            polynomial_mul(p.underlying, r.underlying)))
    polynomial_mul_add_left(p.underlying, q.underlying, r.underlying)
}

/// Wrapped polynomial multiplication distributes over addition on the right.
theorem polyc_semiring_distrib_right_law[R: CommRing](p: PolyC[R], q: PolyC[R], r: PolyC[R]) {
    (p + q) * r = p * r + q * r
} by {
    polyc_typeclass_add_eq_polyc_add(p, q)
    p + q = polyc_add(p, q)
    polyc_typeclass_mul_eq_polyc_mul(p + q, r)
    (p + q) * r = polyc_mul(p + q, r)
    (p + q) * r = polyc_mul(polyc_add(p, q), r)
    polyc_typeclass_add_eq_polyc_add(p * r, q * r)
    p * r + q * r = polyc_add(p * r, q * r)
    polyc_typeclass_mul_eq_polyc_mul(p, r)
    p * r = polyc_mul(p, r)
    polyc_typeclass_mul_eq_polyc_mul(q, r)
    q * r = polyc_mul(q, r)
    polyc_mul(polyc_add(p, q), r) =
        PolyC[R].new(polynomial_mul(polynomial_add(p.underlying, q.underlying), r.underlying))
    polyc_add(polyc_mul(p, r), polyc_mul(q, r)) =
        PolyC[R].new(polynomial_add(polynomial_mul(p.underlying, r.underlying),
            polynomial_mul(q.underlying, r.underlying)))
    polynomial_mul_add_right(p.underlying, q.underlying, r.underlying)
}

/// Multiplying by the wrapped zero on the right gives the wrapped zero.
theorem polyc_semiring_mul_zero_law[R: CommRing](p: PolyC[R]) {
    p * Zero.0[PolyC[R]] = Zero.0[PolyC[R]]
} by {
    polyc_typeclass_zero_eq_polyc_zero[R]
    Zero.0[PolyC[R]] = polyc_zero[R]
    polyc_typeclass_mul_eq_polyc_mul(p, Zero.0[PolyC[R]])
    p * Zero.0[PolyC[R]] = polyc_mul(p, Zero.0[PolyC[R]])
    p * Zero.0[PolyC[R]] = polyc_mul(p, polyc_zero[R])
    polyc_mul(p, polyc_zero[R]) = PolyC[R].new(polynomial_mul(p.underlying, polynomial_zero[R]))
    polynomial_mul_zero_right(p.underlying)
    polyc_mul(p, polyc_zero[R]) = PolyC[R].new(polynomial_zero[R])
    polyc_typeclass_zero_eq_polyc_zero[R]
    Zero.0[PolyC[R]] = polyc_zero[R]
    polyc_zero[R] = PolyC[R].new(polynomial_zero[R])
    Zero.0[PolyC[R]] = PolyC[R].new(polynomial_zero[R])
}

/// Multiplying by the wrapped zero on the left gives the wrapped zero.
theorem polyc_semiring_zero_mul_law[R: CommRing](p: PolyC[R]) {
    Zero.0[PolyC[R]] * p = Zero.0[PolyC[R]]
} by {
    polyc_typeclass_zero_eq_polyc_zero[R]
    Zero.0[PolyC[R]] = polyc_zero[R]
    polyc_typeclass_mul_eq_polyc_mul(Zero.0[PolyC[R]], p)
    Zero.0[PolyC[R]] * p = polyc_mul(Zero.0[PolyC[R]], p)
    Zero.0[PolyC[R]] * p = polyc_mul(polyc_zero[R], p)
    polyc_mul(polyc_zero[R], p) = PolyC[R].new(polynomial_mul(polynomial_zero[R], p.underlying))
    polynomial_mul_zero_left(p.underlying)
    polyc_mul(polyc_zero[R], p) = PolyC[R].new(polynomial_zero[R])
    polyc_typeclass_zero_eq_polyc_zero[R]
    Zero.0[PolyC[R]] = polyc_zero[R]
    polyc_zero[R] = PolyC[R].new(polynomial_zero[R])
    Zero.0[PolyC[R]] = PolyC[R].new(polynomial_zero[R])
}

// -- the typeclass instances -------------------------------------------------

/// Coefficientwise polynomial addition satisfies the exact additive-semigroup law.
theorem polyc_add_semigroup_instance_law[R: CommRing](p: PolyC[R], q: PolyC[R], r: PolyC[R]) {
    Add.add(p, Add.add(q, r)) = Add.add(Add.add(p, q), r)
} by {
    polyc_add_semigroup_law(p, q, r)
}

/// Coefficientwise polynomial addition satisfies the exact commutative law.
theorem polyc_add_comm_semigroup_instance_law[R: CommRing](p: PolyC[R], q: PolyC[R]) {
    Add.add(p, q) = Add.add(q, p)
} by {
    polyc_add_comm_semigroup_law(p, q)
}

/// Wrapped polynomial negation satisfies the exact additive-inverse law.
theorem polyc_add_group_instance_law[R: CommRing](p: PolyC[R]) {
    Add.add(p, Neg.neg(p)) = Zero.0[PolyC[R]]
} by {
    polyc_add_group_inverse_law(p)
}

/// Convolution polynomial multiplication satisfies the exact semigroup law.
theorem polyc_mul_semigroup_instance_law[R: CommRing](p: PolyC[R], q: PolyC[R], r: PolyC[R]) {
    Mul.mul(p, Mul.mul(q, r)) = Mul.mul(Mul.mul(p, q), r)
} by {
    polyc_mul_semigroup_law(p, q, r)
}

/// Convolution polynomial multiplication satisfies the exact commutative law.
theorem polyc_mul_comm_semigroup_instance_law[R: CommRing](p: PolyC[R], q: PolyC[R]) {
    Mul.mul(p, q) = Mul.mul(q, p)
} by {
    polyc_mul_comm_semigroup_law(p, q)
}

/// Polynomial multiplication distributes over addition on the left, exactly.
theorem polyc_semiring_distrib_left_instance_law[R: CommRing](p: PolyC[R], q: PolyC[R], r: PolyC[R]) {
    Mul.mul(p, Add.add(q, r)) = Add.add(Mul.mul(p, q), Mul.mul(p, r))
} by {
    polyc_semiring_distrib_left_law(p, q, r)
}

/// Polynomial multiplication distributes over addition on the right, exactly.
theorem polyc_semiring_distrib_right_instance_law[R: CommRing](p: PolyC[R], q: PolyC[R], r: PolyC[R]) {
    Mul.mul(Add.add(p, q), r) = Add.add(Mul.mul(p, r), Mul.mul(q, r))
} by {
    polyc_semiring_distrib_right_law(p, q, r)
}

/// Wrapped polynomials over a commutative ring form an additive semigroup.
instance PolyC[R: CommRing]: AddSemigroup

/// Wrapped polynomials over a commutative ring form an additive commutative semigroup.
instance PolyC[R: CommRing]: AddCommSemigroup

/// Wrapped polynomials over a commutative ring form an additive monoid.
instance PolyC[R: CommRing]: AddMonoid

/// Wrapped polynomials over a commutative ring form an additive commutative monoid.
instance PolyC[R: CommRing]: AddCommMonoid

/// Wrapped polynomials over a commutative ring form an additive group.
instance PolyC[R: CommRing]: AddGroup

/// Wrapped polynomials over a commutative ring form an additive commutative group.
instance PolyC[R: CommRing]: AddCommGroup

/// Wrapped polynomials over a commutative ring form a multiplicative semigroup.
instance PolyC[R: CommRing]: Semigroup

/// Wrapped polynomials over a commutative ring form a commutative semigroup.
instance PolyC[R: CommRing]: CommSemigroup

/// Wrapped polynomials over a commutative ring form a monoid.
instance PolyC[R: CommRing]: Monoid

/// Wrapped polynomials over a commutative ring form a commutative monoid.
instance PolyC[R: CommRing]: CommMonoid

/// Wrapped polynomials over a commutative ring form a semiring.
instance PolyC[R: CommRing]: Semiring

/// Wrapped polynomials over a commutative ring form a ring.
instance PolyC[R: CommRing]: Ring

/// Wrapped polynomials over a commutative ring form a commutative ring.
instance PolyC[R: CommRing]: CommRing

// ---------------------------------------------------------------------------
// Evaluation of a wrapped polynomial at a ring element
// ---------------------------------------------------------------------------

/// Evaluation of a wrapped polynomial at a ring element.
define polyc_eval[R: CommRing](c: PolyC[R], x: R) -> R {
    polynomial_eval(c.underlying, x)
}

/// Evaluation of a wrapped sum is the sum of evaluations.
theorem polyc_eval_add[R: CommRing](c: PolyC[R], d: PolyC[R], x: R) {
    polyc_eval(polyc_add(c, d), x) = polyc_eval(c, x) + polyc_eval(d, x)
} by {
    polyc_eval(polyc_add(c, d), x) = polynomial_eval(polynomial_add(c.underlying, d.underlying), x)
    polynomial_eval_add(c.underlying, d.underlying, x)
    polynomial_eval(polynomial_add(c.underlying, d.underlying), x) =
        polynomial_eval(c.underlying, x) + polynomial_eval(d.underlying, x)
    polyc_eval(c, x) = polynomial_eval(c.underlying, x)
    polyc_eval(d, x) = polynomial_eval(d.underlying, x)
    polyc_eval(polyc_add(c, d), x) = polyc_eval(c, x) + polyc_eval(d, x)
}

/// Evaluation of a wrapped product is the product of evaluations.
theorem polyc_eval_mul[R: CommRing](c: PolyC[R], d: PolyC[R], x: R) {
    polyc_eval(polyc_mul(c, d), x) = polyc_eval(c, x) * polyc_eval(d, x)
} by {
    polyc_eval(polyc_mul(c, d), x) = polynomial_eval(polynomial_mul(c.underlying, d.underlying), x)
    polynomial_eval_mul(c.underlying, d.underlying, x)
    polynomial_eval(polynomial_mul(c.underlying, d.underlying), x) =
        polynomial_eval(c.underlying, x) * polynomial_eval(d.underlying, x)
    polyc_eval(c, x) = polynomial_eval(c.underlying, x)
    polyc_eval(d, x) = polynomial_eval(d.underlying, x)
    polyc_eval(polyc_mul(c, d), x) = polyc_eval(c, x) * polyc_eval(d, x)
}

/// Evaluation of the wrapped negative is the negative of the evaluation.
theorem polyc_eval_neg[R: CommRing](c: PolyC[R], x: R) {
    polyc_eval(polyc_neg(c), x) = -polyc_eval(c, x)
} by {
    polyc_eval(polyc_neg(c), x) = polynomial_eval(polynomial_neg(c.underlying), x)
    polynomial_neg(c.underlying) = c.underlying.neg
    polyc_eval(polyc_neg(c), x) = polynomial_eval(c.underlying.neg, x)
    polynomial_eval_neg(c.underlying, x)
    polynomial_eval(c.underlying.neg, x) = -polynomial_eval(c.underlying, x)
    polyc_eval(c, x) = polynomial_eval(c.underlying, x)
    polyc_eval(polyc_neg(c), x) = -polyc_eval(c, x)
}

/// Evaluation of a wrapped difference is the difference of evaluations.
theorem polyc_eval_sub[R: CommRing](c: PolyC[R], d: PolyC[R], x: R) {
    polyc_eval(c - d, x) = polyc_eval(c, x) - polyc_eval(d, x)
} by {
    polyc_eval(c - d, x) = polyc_eval(c + -d, x)
    polyc_eval_add(c, -d, x)
    polyc_eval(c + -d, x) = polyc_eval(c, x) + polyc_eval(-d, x)
    polyc_typeclass_neg_eq_polyc_neg(d)
    -d = polyc_neg(d)
    polyc_eval(c - d, x) = polyc_eval(c, x) + polyc_eval(polyc_neg(d), x)
    polyc_eval_neg(d, x)
    polyc_eval(polyc_neg(d), x) = -polyc_eval(d, x)
    polyc_eval(c - d, x) = polyc_eval(c, x) + -polyc_eval(d, x)
    polyc_eval(c, x) - polyc_eval(d, x) = polyc_eval(c, x) + -polyc_eval(d, x)
    polyc_eval(c - d, x) = polyc_eval(c, x) - polyc_eval(d, x)
}

/// Evaluation of the wrapped zero polynomial is zero.
theorem polyc_eval_zero[R: CommRing](x: R) {
    polyc_eval(polyc_zero[R], x) = R.0
} by {
    polyc_eval(polyc_zero[R], x) = polynomial_eval(polynomial_zero[R], x)
    polynomial_eval_zero[R](x)
    polynomial_eval(polynomial_zero[R], x) = R.0
}

/// Evaluation of the wrapped one polynomial is one.
theorem polyc_eval_one[R: CommRing](x: R) {
    polyc_eval(polyc_one[R], x) = R.1
} by {
    polyc_eval(polyc_one[R], x) = polynomial_eval(polynomial_one[R], x)
    polynomial_eval_one[R](x)
    polynomial_eval(polynomial_one[R], x) = R.1
}

/// Evaluation of a wrapped constant polynomial is the constant.
theorem polyc_eval_constant[R: CommRing](r: R, x: R) {
    polyc_eval(polyc_constant(r), x) = r
} by {
    polyc_eval(polyc_constant(r), x) = polynomial_eval(polynomial_constant(r), x)
    polynomial_eval_constant(r, x)
    polynomial_eval(polynomial_constant(r), x) = r
}

/// The monomial `X` is supported below degree two.
theorem monomial_x_support_bounded[S: Semiring] {
    polynomial_support_bounded_by(polynomial_monomial(Nat.1, S.1), Nat.2)
} by {
    polynomial_support_bounded_by(polynomial_monomial(Nat.1, S.1), Nat.2) =
        coeff_zero_from(polynomial_monomial(Nat.1, S.1).coeff, Nat.2)
    coeff_zero_from(polynomial_monomial(Nat.1, S.1).coeff, Nat.2) =
        forall(k: Nat) { coeff_zero_from_at(polynomial_monomial(Nat.1, S.1).coeff, Nat.2, k) }
    forall(k: Nat) {
        coeff_zero_from_at(polynomial_monomial(Nat.1, S.1).coeff, Nat.2, k) =
            (not k < Nat.2 implies coeff_zero_at(polynomial_monomial(Nat.1, S.1).coeff, k))
        if not k < Nat.2 {
            if k = Nat.1 {
                lt_suc(Nat.1)
                Nat.1 < Nat.2
                k < Nat.2
                false
            }
            k != Nat.1
            polynomial_monomial_coeff_of_ne(Nat.1, S.1, k)
            polynomial_monomial(Nat.1, S.1).coeff(k) = S.0
            coeff_zero_at(polynomial_monomial(Nat.1, S.1).coeff, k)
        }
        not k < Nat.2 implies coeff_zero_at(polynomial_monomial(Nat.1, S.1).coeff, k)
        coeff_zero_from_at(polynomial_monomial(Nat.1, S.1).coeff, Nat.2, k)
    }
    coeff_zero_from(polynomial_monomial(Nat.1, S.1).coeff, Nat.2)
    polynomial_support_bounded_by(polynomial_monomial(Nat.1, S.1), Nat.2)
}

/// Evaluation of the monomial `X` (coefficient one, degree one) at a point
/// is the point itself.
theorem monomial_x_eval[S: CommRing](x: S) {
    polynomial_eval(polynomial_monomial(Nat.1, S.1), x) = x
} by {
    monomial_x_support_bounded[S]
    polynomial_support_bounded_by(polynomial_monomial(Nat.1, S.1), Nat.2)
    polynomial_eval_eq_eval_bound_of_support_bounded(polynomial_monomial(Nat.1, S.1), x, Nat.2)
    polynomial_eval(polynomial_monomial(Nat.1, S.1), x) =
        polynomial_eval_bound(polynomial_monomial(Nat.1, S.1), x, Nat.2)
    polynomial_eval_bound_eq_coeff_eval(polynomial_monomial(Nat.1, S.1), x, Nat.2)
    polynomial_eval_bound(polynomial_monomial(Nat.1, S.1), x, Nat.2) =
        coeff_eval(polynomial_monomial(Nat.1, S.1).coeff, x, Nat.2)
    coeff_eval(polynomial_monomial(Nat.1, S.1).coeff, x, Nat.2) =
        polynomial_monomial(Nat.1, S.1).coeff(Nat.0) +
        x * coeff_eval(coeff_tail(polynomial_monomial(Nat.1, S.1).coeff), x, Nat.1)
    Nat.0 != Nat.1
    polynomial_monomial_coeff_of_ne(Nat.1, S.1, Nat.0)
    polynomial_monomial(Nat.1, S.1).coeff(Nat.0) = S.0
    coeff_eval(coeff_tail(polynomial_monomial(Nat.1, S.1).coeff), x, Nat.1) =
        coeff_tail(polynomial_monomial(Nat.1, S.1).coeff)(Nat.0) +
        x * coeff_eval(coeff_tail(coeff_tail(polynomial_monomial(Nat.1, S.1).coeff)), x, Nat.0)
    coeff_tail(polynomial_monomial(Nat.1, S.1).coeff)(Nat.0) =
        polynomial_monomial(Nat.1, S.1).coeff(Nat.1)
    polynomial_monomial_coeff_self(Nat.1, S.1)
    polynomial_monomial(Nat.1, S.1).coeff(Nat.1) = S.1
    coeff_eval(coeff_tail(coeff_tail(polynomial_monomial(Nat.1, S.1).coeff)), x, Nat.0) = S.0
    coeff_eval(polynomial_monomial(Nat.1, S.1).coeff, x, Nat.2) = S.0 + x * (S.1 + x * S.0)
    S.0 + x * (S.1 + x * S.0) = x
    polynomial_eval(polynomial_monomial(Nat.1, S.1), x) = x
}

/// Evaluation of the wrapped polynomial `X` at `x` is `x`.
theorem polyc_eval_x[R: CommRing](x: R) {
    polyc_eval(polyc_x[R], x) = x
} by {
    polyc_eval(polyc_x[R], x) = polynomial_eval(polyc_x[R].underlying, x)
    polyc_x[R].underlying = polynomial_monomial(Nat.1, R.1)
    polyc_eval(polyc_x[R], x) = polynomial_eval(polynomial_monomial(Nat.1, R.1), x)
    monomial_x_eval[R](x)
    polynomial_eval(polynomial_monomial(Nat.1, R.1), x) = x
}

// ---------------------------------------------------------------------------
// Bivariate polynomials and their evaluation at points of the plane
// ---------------------------------------------------------------------------

/// A bivariate polynomial over `R`: a polynomial in `Y` with coefficients in
/// the wrapped polynomials in `X`.
structure Poly2[R: CommRing] {
    /// The underlying polynomial in `Y` over `PolyC[R]`.
    underlying: Polynomial[PolyC[R]]
}

/// Addition of bivariate polynomials.
define poly2_add[R: CommRing](p: Poly2[R], q: Poly2[R]) -> Poly2[R] {
    Poly2[R].new(polynomial_add(p.underlying, q.underlying))
}

/// Multiplication of bivariate polynomials.
define poly2_mul[R: CommRing](p: Poly2[R], q: Poly2[R]) -> Poly2[R] {
    Poly2[R].new(polynomial_mul(p.underlying, q.underlying))
}

/// Negation of a bivariate polynomial.
define poly2_neg[R: CommRing](p: Poly2[R]) -> Poly2[R] {
    Poly2[R].new(polynomial_neg(p.underlying))
}

/// Subtraction of bivariate polynomials.
define poly2_sub[R: CommRing](p: Poly2[R], q: Poly2[R]) -> Poly2[R] {
    Poly2[R].new(p.underlying.sub(q.underlying))
}

/// The zero bivariate polynomial.
let biv_zero[R: CommRing]: Poly2[R] = Poly2[R].new(polynomial_zero[PolyC[R]])

/// The one bivariate polynomial.
let biv_one[R: CommRing]: Poly2[R] = Poly2[R].new(polynomial_one[PolyC[R]])

/// The coordinate polynomial `X` (constant in `Y`).
let biv_x[R: CommRing]: Poly2[R] = Poly2[R].new(polynomial_constant[PolyC[R]](polyc_x[R]))

/// The coordinate polynomial `Y`.
let biv_y[R: CommRing]: Poly2[R] = Poly2[R].new(polynomial_monomial[PolyC[R]](Nat.1, polyc_one[R]))

/// A bivariate polynomial is determined by its underlying polynomial.
theorem poly2_ext[R: CommRing](p: Poly2[R], q: Poly2[R]) {
    p.underlying = q.underlying implies p = q
}

/// Multiplying a bivariate polynomial by one on the left gives it back.
theorem poly2_mul_one_left[R: CommRing](p: Poly2[R]) {
    poly2_mul(biv_one[R], p) = p
} by {
    poly2_mul(biv_one[R], p).underlying = polynomial_mul(biv_one[R].underlying, p.underlying)
    biv_one[R].underlying = polynomial_one[PolyC[R]]
    poly2_mul(biv_one[R], p).underlying = polynomial_mul(polynomial_one[PolyC[R]], p.underlying)
    polynomial_mul_one_left[PolyC[R]](p.underlying)
    polynomial_mul(polynomial_one[PolyC[R]], p.underlying) = p.underlying
    poly2_mul(biv_one[R], p).underlying = p.underlying
    poly2_ext(poly2_mul(biv_one[R], p), p)
}

/// The bivariate polynomial `y = x^2`, i.e. `Y - X * X` (the parabola).
let parabola_poly[R: CommRing]: Poly2[R] =
    poly2_sub(biv_y[R], poly2_mul(biv_x[R], biv_x[R]))

/// The bivariate polynomial `X^2 + Y^2 - 1` (the unit circle).
let circle_poly[R: CommRing]: Poly2[R] =
    poly2_sub(poly2_add(poly2_mul(biv_x[R], biv_x[R]), poly2_mul(biv_y[R], biv_y[R])), biv_one[R])

/// Evaluation of a bivariate polynomial at `(x, y)`: substitute `y` for `Y`,
/// then evaluate the resulting polynomial in `X` at `x`.
define poly2_eval[R: CommRing](p: Poly2[R], x: R, y: R) -> R {
    polyc_eval(polynomial_eval(p.underlying, polyc_constant(y)), x)
}

/// Evaluation of a bivariate sum is the sum of the evaluations.
theorem poly2_eval_add[R: CommRing](p: Poly2[R], q: Poly2[R], x: R, y: R) {
    poly2_eval(poly2_add(p, q), x, y) = poly2_eval(p, x, y) + poly2_eval(q, x, y)
} by {
    poly2_eval(poly2_add(p, q), x, y) =
        polyc_eval(polynomial_eval(poly2_add(p, q).underlying, polyc_constant(y)), x)
    poly2_add(p, q).underlying = polynomial_add(p.underlying, q.underlying)
    poly2_eval(poly2_add(p, q), x, y) =
        polyc_eval(polynomial_eval(polynomial_add(p.underlying, q.underlying), polyc_constant(y)), x)
    polynomial_eval_add[PolyC[R]](p.underlying, q.underlying, polyc_constant(y))
    polynomial_eval(polynomial_add(p.underlying, q.underlying), polyc_constant(y)) =
        polynomial_eval(p.underlying, polyc_constant(y)) + polynomial_eval(q.underlying, polyc_constant(y))
    poly2_eval(poly2_add(p, q), x, y) =
        polyc_eval(polynomial_eval(p.underlying, polyc_constant(y)) +
            polynomial_eval(q.underlying, polyc_constant(y)), x)
    polyc_typeclass_add_eq_polyc_add(polynomial_eval(p.underlying, polyc_constant(y)),
        polynomial_eval(q.underlying, polyc_constant(y)))
    polynomial_eval(p.underlying, polyc_constant(y)) + polynomial_eval(q.underlying, polyc_constant(y)) =
        polyc_add(polynomial_eval(p.underlying, polyc_constant(y)),
            polynomial_eval(q.underlying, polyc_constant(y)))
    poly2_eval(poly2_add(p, q), x, y) =
        polyc_eval(polyc_add(polynomial_eval(p.underlying, polyc_constant(y)),
            polynomial_eval(q.underlying, polyc_constant(y))), x)
    polyc_eval_add(polynomial_eval(p.underlying, polyc_constant(y)),
        polynomial_eval(q.underlying, polyc_constant(y)), x)
    polyc_eval(polyc_add(polynomial_eval(p.underlying, polyc_constant(y)),
        polynomial_eval(q.underlying, polyc_constant(y))), x) =
        polyc_eval(polynomial_eval(p.underlying, polyc_constant(y)), x) +
        polyc_eval(polynomial_eval(q.underlying, polyc_constant(y)), x)
    poly2_eval(poly2_add(p, q), x, y) =
        polyc_eval(polynomial_eval(p.underlying, polyc_constant(y)), x) +
        polyc_eval(polynomial_eval(q.underlying, polyc_constant(y)), x)
    poly2_eval(p, x, y) = polyc_eval(polynomial_eval(p.underlying, polyc_constant(y)), x)
    poly2_eval(q, x, y) = polyc_eval(polynomial_eval(q.underlying, polyc_constant(y)), x)
    poly2_eval(poly2_add(p, q), x, y) = poly2_eval(p, x, y) + poly2_eval(q, x, y)
}

/// Evaluation of a bivariate product is the product of the evaluations.
theorem poly2_eval_mul[R: CommRing](p: Poly2[R], q: Poly2[R], x: R, y: R) {
    poly2_eval(poly2_mul(p, q), x, y) = poly2_eval(p, x, y) * poly2_eval(q, x, y)
} by {
    poly2_eval(poly2_mul(p, q), x, y) =
        polyc_eval(polynomial_eval(poly2_mul(p, q).underlying, polyc_constant(y)), x)
    poly2_mul(p, q).underlying = polynomial_mul(p.underlying, q.underlying)
    poly2_eval(poly2_mul(p, q), x, y) =
        polyc_eval(polynomial_eval(polynomial_mul(p.underlying, q.underlying), polyc_constant(y)), x)
    polynomial_eval_mul[PolyC[R]](p.underlying, q.underlying, polyc_constant(y))
    polynomial_eval(polynomial_mul(p.underlying, q.underlying), polyc_constant(y)) =
        polynomial_eval(p.underlying, polyc_constant(y)) * polynomial_eval(q.underlying, polyc_constant(y))
    poly2_eval(poly2_mul(p, q), x, y) =
        polyc_eval(polynomial_eval(p.underlying, polyc_constant(y)) *
            polynomial_eval(q.underlying, polyc_constant(y)), x)
    polyc_typeclass_mul_eq_polyc_mul(polynomial_eval(p.underlying, polyc_constant(y)),
        polynomial_eval(q.underlying, polyc_constant(y)))
    polynomial_eval(p.underlying, polyc_constant(y)) * polynomial_eval(q.underlying, polyc_constant(y)) =
        polyc_mul(polynomial_eval(p.underlying, polyc_constant(y)),
            polynomial_eval(q.underlying, polyc_constant(y)))
    poly2_eval(poly2_mul(p, q), x, y) =
        polyc_eval(polyc_mul(polynomial_eval(p.underlying, polyc_constant(y)),
            polynomial_eval(q.underlying, polyc_constant(y))), x)
    polyc_eval_mul(polynomial_eval(p.underlying, polyc_constant(y)),
        polynomial_eval(q.underlying, polyc_constant(y)), x)
    polyc_eval(polyc_mul(polynomial_eval(p.underlying, polyc_constant(y)),
        polynomial_eval(q.underlying, polyc_constant(y))), x) =
        polyc_eval(polynomial_eval(p.underlying, polyc_constant(y)), x) *
        polyc_eval(polynomial_eval(q.underlying, polyc_constant(y)), x)
    poly2_eval(poly2_mul(p, q), x, y) =
        polyc_eval(polynomial_eval(p.underlying, polyc_constant(y)), x) *
        polyc_eval(polynomial_eval(q.underlying, polyc_constant(y)), x)
    poly2_eval(p, x, y) = polyc_eval(polynomial_eval(p.underlying, polyc_constant(y)), x)
    poly2_eval(q, x, y) = polyc_eval(polynomial_eval(q.underlying, polyc_constant(y)), x)
    poly2_eval(poly2_mul(p, q), x, y) = poly2_eval(p, x, y) * poly2_eval(q, x, y)
}

/// Evaluation of a bivariate negation is the negative of the evaluation.
theorem poly2_eval_neg[R: CommRing](p: Poly2[R], x: R, y: R) {
    poly2_eval(poly2_neg(p), x, y) = -poly2_eval(p, x, y)
} by {
    poly2_eval(poly2_neg(p), x, y) =
        polyc_eval(polynomial_eval(poly2_neg(p).underlying, polyc_constant(y)), x)
    poly2_neg(p).underlying = polynomial_neg(p.underlying)
    poly2_eval(poly2_neg(p), x, y) =
        polyc_eval(polynomial_eval(polynomial_neg(p.underlying), polyc_constant(y)), x)
    polynomial_neg(p.underlying) = p.underlying.neg
    poly2_eval(poly2_neg(p), x, y) =
        polyc_eval(polynomial_eval(p.underlying.neg, polyc_constant(y)), x)
    polynomial_eval_neg[PolyC[R]](p.underlying, polyc_constant(y))
    polynomial_eval(p.underlying.neg, polyc_constant(y)) =
        -polynomial_eval(p.underlying, polyc_constant(y))
    poly2_eval(poly2_neg(p), x, y) =
        polyc_eval(-polynomial_eval(p.underlying, polyc_constant(y)), x)
    polyc_typeclass_neg_eq_polyc_neg(polynomial_eval(p.underlying, polyc_constant(y)))
    -polynomial_eval(p.underlying, polyc_constant(y)) =
        polyc_neg(polynomial_eval(p.underlying, polyc_constant(y)))
    poly2_eval(poly2_neg(p), x, y) =
        polyc_eval(polyc_neg(polynomial_eval(p.underlying, polyc_constant(y))), x)
    polyc_eval_neg(polynomial_eval(p.underlying, polyc_constant(y)), x)
    polyc_eval(polyc_neg(polynomial_eval(p.underlying, polyc_constant(y))), x) =
        -polyc_eval(polynomial_eval(p.underlying, polyc_constant(y)), x)
    poly2_eval(poly2_neg(p), x, y) =
        -polyc_eval(polynomial_eval(p.underlying, polyc_constant(y)), x)
    poly2_eval(p, x, y) = polyc_eval(polynomial_eval(p.underlying, polyc_constant(y)), x)
    poly2_eval(poly2_neg(p), x, y) = -poly2_eval(p, x, y)
}

/// Evaluation of a bivariate difference is the difference of the evaluations.
theorem poly2_eval_sub[R: CommRing](p: Poly2[R], q: Poly2[R], x: R, y: R) {
    poly2_eval(poly2_sub(p, q), x, y) = poly2_eval(p, x, y) - poly2_eval(q, x, y)
} by {
    poly2_eval(poly2_sub(p, q), x, y) =
        polyc_eval(polynomial_eval(poly2_sub(p, q).underlying, polyc_constant(y)), x)
    poly2_sub(p, q).underlying = p.underlying.sub(q.underlying)
    poly2_eval(poly2_sub(p, q), x, y) =
        polyc_eval(polynomial_eval(p.underlying.sub(q.underlying), polyc_constant(y)), x)
    polynomial_eval_sub[PolyC[R]](p.underlying, q.underlying, polyc_constant(y))
    polynomial_eval(p.underlying.sub(q.underlying), polyc_constant(y)) =
        polynomial_eval(p.underlying, polyc_constant(y)) - polynomial_eval(q.underlying, polyc_constant(y))
    poly2_eval(poly2_sub(p, q), x, y) =
        polyc_eval(polynomial_eval(p.underlying, polyc_constant(y)) -
            polynomial_eval(q.underlying, polyc_constant(y)), x)
    polyc_eval_sub(polynomial_eval(p.underlying, polyc_constant(y)),
        polynomial_eval(q.underlying, polyc_constant(y)), x)
    polyc_eval(polynomial_eval(p.underlying, polyc_constant(y)) -
        polynomial_eval(q.underlying, polyc_constant(y)), x) =
        polyc_eval(polynomial_eval(p.underlying, polyc_constant(y)), x) -
        polyc_eval(polynomial_eval(q.underlying, polyc_constant(y)), x)
    poly2_eval(poly2_sub(p, q), x, y) =
        polyc_eval(polynomial_eval(p.underlying, polyc_constant(y)), x) -
        polyc_eval(polynomial_eval(q.underlying, polyc_constant(y)), x)
    poly2_eval(p, x, y) = polyc_eval(polynomial_eval(p.underlying, polyc_constant(y)), x)
    poly2_eval(q, x, y) = polyc_eval(polynomial_eval(q.underlying, polyc_constant(y)), x)
    poly2_eval(poly2_sub(p, q), x, y) = poly2_eval(p, x, y) - poly2_eval(q, x, y)
}

/// Evaluation of the zero bivariate polynomial is zero.
theorem poly2_eval_zero[R: CommRing](x: R, y: R) {
    poly2_eval(biv_zero[R], x, y) = R.0
} by {
    poly2_eval(biv_zero[R], x, y) =
        polyc_eval(polynomial_eval(biv_zero[R].underlying, polyc_constant(y)), x)
    biv_zero[R].underlying = polynomial_zero[PolyC[R]]
    poly2_eval(biv_zero[R], x, y) =
        polyc_eval(polynomial_eval(polynomial_zero[PolyC[R]], polyc_constant(y)), x)
    polynomial_eval_zero[PolyC[R]](polyc_constant(y))
    polynomial_eval(polynomial_zero[PolyC[R]], polyc_constant(y)) = Zero.0[PolyC[R]]
    poly2_eval(biv_zero[R], x, y) = polyc_eval(Zero.0[PolyC[R]], x)
    polyc_typeclass_zero_eq_polyc_zero[R]
    Zero.0[PolyC[R]] = polyc_zero[R]
    poly2_eval(biv_zero[R], x, y) = polyc_eval(polyc_zero[R], x)
    polyc_eval_zero[R](x)
    polyc_eval(polyc_zero[R], x) = R.0
}

/// Evaluation of the one bivariate polynomial is one.
theorem poly2_eval_one[R: CommRing](x: R, y: R) {
    poly2_eval(biv_one[R], x, y) = R.1
} by {
    poly2_eval(biv_one[R], x, y) =
        polyc_eval(polynomial_eval(biv_one[R].underlying, polyc_constant(y)), x)
    biv_one[R].underlying = polynomial_one[PolyC[R]]
    poly2_eval(biv_one[R], x, y) =
        polyc_eval(polynomial_eval(polynomial_one[PolyC[R]], polyc_constant(y)), x)
    polynomial_eval_one[PolyC[R]](polyc_constant(y))
    polynomial_eval(polynomial_one[PolyC[R]], polyc_constant(y)) = One.1[PolyC[R]]
    poly2_eval(biv_one[R], x, y) = polyc_eval(One.1[PolyC[R]], x)
    polyc_typeclass_one_eq_polyc_one[R]
    One.1[PolyC[R]] = polyc_one[R]
    poly2_eval(biv_one[R], x, y) = polyc_eval(polyc_one[R], x)
    polyc_eval_one[R](x)
    polyc_eval(polyc_one[R], x) = R.1
}

/// Evaluation of the coordinate polynomial `X` at `(x, y)` is `x`.
theorem poly2_eval_x[R: CommRing](x: R, y: R) {
    poly2_eval(biv_x[R], x, y) = x
} by {
    poly2_eval(biv_x[R], x, y) =
        polyc_eval(polynomial_eval(biv_x[R].underlying, polyc_constant(y)), x)
    biv_x[R].underlying = polynomial_constant[PolyC[R]](polyc_x[R])
    poly2_eval(biv_x[R], x, y) =
        polyc_eval(polynomial_eval(polynomial_constant[PolyC[R]](polyc_x[R]), polyc_constant(y)), x)
    polynomial_eval_constant[PolyC[R]](polyc_x[R], polyc_constant(y))
    polynomial_eval(polynomial_constant[PolyC[R]](polyc_x[R]), polyc_constant(y)) = polyc_x[R]
    poly2_eval(biv_x[R], x, y) = polyc_eval(polyc_x[R], x)
    polyc_eval(polyc_x[R], x) = polynomial_eval(polynomial_monomial(Nat.1, R.1), x)
    polyc_eval_x[R](x)
    polyc_eval(polyc_x[R], x) = x
    poly2_eval(biv_x[R], x, y) = x
}

/// Evaluation of the coordinate polynomial `Y` at `(x, y)` is `y`.
theorem poly2_eval_y[R: CommRing](x: R, y: R) {
    poly2_eval(biv_y[R], x, y) = y
} by {
    poly2_eval(biv_y[R], x, y) =
        polyc_eval(polynomial_eval(biv_y[R].underlying, polyc_constant(y)), x)
    biv_y[R].underlying = polynomial_monomial[PolyC[R]](Nat.1, polyc_one[R])
    poly2_eval(biv_y[R], x, y) =
        polyc_eval(polynomial_eval(polynomial_monomial[PolyC[R]](Nat.1, polyc_one[R]), polyc_constant(y)), x)
    polyc_typeclass_one_eq_polyc_one[R]
    One.1[PolyC[R]] = polyc_one[R]
    polynomial_monomial[PolyC[R]](Nat.1, polyc_one[R]) =
        polynomial_monomial[PolyC[R]](Nat.1, One.1[PolyC[R]])
    monomial_x_eval[PolyC[R]](polyc_constant(y))
    polynomial_eval(polynomial_monomial[PolyC[R]](Nat.1, One.1[PolyC[R]]), polyc_constant(y)) =
        polyc_constant(y)
    polynomial_eval(polynomial_monomial[PolyC[R]](Nat.1, polyc_one[R]), polyc_constant(y)) =
        polyc_constant(y)
    poly2_eval(biv_y[R], x, y) = polyc_eval(polyc_constant(y), x)
    polyc_eval_constant(y, x)
    polyc_eval(polyc_constant(y), x) = y
    poly2_eval(biv_y[R], x, y) = y
}

/// The evaluation of the parabola polynomial is `y - x * x`.
theorem poly2_eval_parabola[R: CommRing](x: R, y: R) {
    poly2_eval(parabola_poly[R], x, y) = y - x * x
} by {
    parabola_poly[R] = poly2_sub(biv_y[R], poly2_mul(biv_x[R], biv_x[R]))
    poly2_eval(parabola_poly[R], x, y) =
        poly2_eval(poly2_sub(biv_y[R], poly2_mul(biv_x[R], biv_x[R])), x, y)
    poly2_eval_sub(biv_y[R], poly2_mul(biv_x[R], biv_x[R]), x, y)
    poly2_eval(poly2_sub(biv_y[R], poly2_mul(biv_x[R], biv_x[R])), x, y) =
        poly2_eval(biv_y[R], x, y) - poly2_eval(poly2_mul(biv_x[R], biv_x[R]), x, y)
    poly2_eval(parabola_poly[R], x, y) =
        poly2_eval(biv_y[R], x, y) - poly2_eval(poly2_mul(biv_x[R], biv_x[R]), x, y)
    poly2_eval_y[R](x, y)
    poly2_eval(biv_y[R], x, y) = y
    poly2_eval_mul(biv_x[R], biv_x[R], x, y)
    poly2_eval(poly2_mul(biv_x[R], biv_x[R]), x, y) = poly2_eval(biv_x[R], x, y) * poly2_eval(biv_x[R], x, y)
    poly2_eval_x[R](x, y)
    poly2_eval(biv_x[R], x, y) = x
    poly2_eval(parabola_poly[R], x, y) = y - x * x
}

/// The evaluation of the unit circle polynomial is `x * x + y * y - 1`.
theorem poly2_eval_circle[R: CommRing](x: R, y: R) {
    poly2_eval(circle_poly[R], x, y) = x * x + y * y - R.1
} by {
    circle_poly[R] = poly2_sub(poly2_add(poly2_mul(biv_x[R], biv_x[R]),
        poly2_mul(biv_y[R], biv_y[R])), biv_one[R])
    poly2_eval(circle_poly[R], x, y) =
        poly2_eval(poly2_sub(poly2_add(poly2_mul(biv_x[R], biv_x[R]),
            poly2_mul(biv_y[R], biv_y[R])), biv_one[R]), x, y)
    poly2_eval_sub(poly2_add(poly2_mul(biv_x[R], biv_x[R]), poly2_mul(biv_y[R], biv_y[R])),
        biv_one[R], x, y)
    poly2_eval(poly2_sub(poly2_add(poly2_mul(biv_x[R], biv_x[R]),
        poly2_mul(biv_y[R], biv_y[R])), biv_one[R]), x, y) =
        poly2_eval(poly2_add(poly2_mul(biv_x[R], biv_x[R]), poly2_mul(biv_y[R], biv_y[R])), x, y) -
        poly2_eval(biv_one[R], x, y)
    poly2_eval(circle_poly[R], x, y) =
        poly2_eval(poly2_add(poly2_mul(biv_x[R], biv_x[R]), poly2_mul(biv_y[R], biv_y[R])), x, y) -
        poly2_eval(biv_one[R], x, y)
    poly2_eval_add(poly2_mul(biv_x[R], biv_x[R]), poly2_mul(biv_y[R], biv_y[R]), x, y)
    poly2_eval(poly2_add(poly2_mul(biv_x[R], biv_x[R]), poly2_mul(biv_y[R], biv_y[R])), x, y) =
        poly2_eval(poly2_mul(biv_x[R], biv_x[R]), x, y) +
        poly2_eval(poly2_mul(biv_y[R], biv_y[R]), x, y)
    poly2_eval(circle_poly[R], x, y) =
        poly2_eval(poly2_mul(biv_x[R], biv_x[R]), x, y) +
        poly2_eval(poly2_mul(biv_y[R], biv_y[R]), x, y) - poly2_eval(biv_one[R], x, y)
    poly2_eval_mul(biv_x[R], biv_x[R], x, y)
    poly2_eval(poly2_mul(biv_x[R], biv_x[R]), x, y) = poly2_eval(biv_x[R], x, y) * poly2_eval(biv_x[R], x, y)
    poly2_eval_mul(biv_y[R], biv_y[R], x, y)
    poly2_eval(poly2_mul(biv_y[R], biv_y[R]), x, y) = poly2_eval(biv_y[R], x, y) * poly2_eval(biv_y[R], x, y)
    poly2_eval_x[R](x, y)
    poly2_eval(biv_x[R], x, y) = x
    poly2_eval_y[R](x, y)
    poly2_eval(biv_y[R], x, y) = y
    poly2_eval_one[R](x, y)
    poly2_eval(biv_one[R], x, y) = R.1
    poly2_eval(circle_poly[R], x, y) = x * x + y * y - R.1
}

// ---------------------------------------------------------------------------
// Affine varieties: the zero set of a family of bivariate polynomials
// ---------------------------------------------------------------------------

/// A point of the affine plane over `R`.
structure PlanePoint[R] {
    /// The first coordinate.
    x: R

    /// The second coordinate.
    y: R
}

/// True if a point is a zero of a bivariate polynomial.
define point_in_zero_set[R: CommRing](f: Poly2[R], p: PlanePoint[R]) -> Bool {
    poly2_eval(f, p.x, p.y) = R.0
}

/// A point vanishes on `f` exactly when the evaluation of `f` at the point
/// is zero.
theorem point_in_zero_set_unfold[R: CommRing](f: Poly2[R], p: PlanePoint[R]) {
    point_in_zero_set(f, p) = (poly2_eval(f, p.x, p.y) = R.0)
} by {
    point_in_zero_set(f, p) = (poly2_eval(f, p.x, p.y) = R.0)
}

/// The first coordinate of a constructed point is the first argument.
theorem plane_point_new_x[R](a: R, b: R) {
    PlanePoint[R].new(a, b).x = a
} by {
    PlanePoint[R].new(a, b).x = a
}

/// The second coordinate of a constructed point is the second argument.
theorem plane_point_new_y[R](a: R, b: R) {
    PlanePoint[R].new(a, b).y = b
} by {
    PlanePoint[R].new(a, b).y = b
}

/// The zero set (affine variety) of a set of bivariate polynomials:
/// all points at which every polynomial of the set vanishes.
define in_variety[R: CommRing](s: Set[Poly2[R]], p: PlanePoint[R]) -> Bool {
    forall(f: Poly2[R]) {
        s.contains(f) implies point_in_zero_set(f, p)
    }
}

/// The zero set of a single bivariate polynomial, as a set of points.
define zero_set[R: CommRing](f: Poly2[R]) -> Set[PlanePoint[R]] {
    Set[PlanePoint[R]].new(function(p: PlanePoint[R]) { point_in_zero_set(f, p) })
}

/// Two, written as `1 + 1` in any field.
let field_two[F: Field]: F = F.1 + F.1

/// Four, written as `1 + 1 + 1 + 1` in any field.
let field_four[F: Field]: F = F.1 + F.1 + F.1 + F.1

/// `4 = 2^2` in any field.
theorem field_two_squared_is_four[F: Field] {
    field_two[F] * field_two[F] = field_four[F]
} by {
    field_two[F] = F.1 + F.1
    field_four[F] = F.1 + F.1 + F.1 + F.1
    (F.1 + F.1) * (F.1 + F.1) = F.1 + F.1 + F.1 + F.1
}

// -- (a) conic sections -------------------------------------------------------

/// The parabola `y = x^2` contains the point (2, 4).
theorem parabola_point_two_four[F: Field] {
    point_in_zero_set(parabola_poly[F], PlanePoint[F].new(field_two[F], field_four[F]))
} by {
    point_in_zero_set_unfold(parabola_poly[F], PlanePoint[F].new(field_two[F], field_four[F]))
    point_in_zero_set(parabola_poly[F], PlanePoint[F].new(field_two[F], field_four[F])) =
        (poly2_eval(parabola_poly[F], PlanePoint[F].new(field_two[F], field_four[F]).x,
            PlanePoint[F].new(field_two[F], field_four[F]).y) = F.0)
    plane_point_new_x(field_two[F], field_four[F])
    PlanePoint[F].new(field_two[F], field_four[F]).x = field_two[F]
    plane_point_new_y(field_two[F], field_four[F])
    PlanePoint[F].new(field_two[F], field_four[F]).y = field_four[F]
    point_in_zero_set(parabola_poly[F], PlanePoint[F].new(field_two[F], field_four[F])) =
        (poly2_eval(parabola_poly[F], field_two[F], field_four[F]) = F.0)
    poly2_eval_parabola[F](field_two[F], field_four[F])
    poly2_eval(parabola_poly[F], field_two[F], field_four[F]) =
        field_four[F] - field_two[F] * field_two[F]
    field_two_squared_is_four[F]
    field_two[F] * field_two[F] = field_four[F]
    inverse_left[F](field_four[F])
    -field_four[F] + field_four[F] = F.0
    field_four[F] + -field_four[F] = F.0
    field_four[F] - field_two[F] * field_two[F] = field_four[F] + -field_four[F]
    field_four[F] - field_two[F] * field_two[F] = F.0
    poly2_eval(parabola_poly[F], field_two[F], field_four[F]) = F.0
    point_in_zero_set(parabola_poly[F], PlanePoint[F].new(field_two[F], field_four[F]))
}

/// The unit circle contains the point (1, 0).
theorem circle_point_one_zero[F: Field] {
    point_in_zero_set(circle_poly[F], PlanePoint[F].new(F.1, F.0))
} by {
    point_in_zero_set_unfold(circle_poly[F], PlanePoint[F].new(F.1, F.0))
    point_in_zero_set(circle_poly[F], PlanePoint[F].new(F.1, F.0)) =
        (poly2_eval(circle_poly[F], PlanePoint[F].new(F.1, F.0).x,
            PlanePoint[F].new(F.1, F.0).y) = F.0)
    plane_point_new_x(F.1, F.0)
    PlanePoint[F].new(F.1, F.0).x = F.1
    plane_point_new_y(F.1, F.0)
    PlanePoint[F].new(F.1, F.0).y = F.0
    point_in_zero_set(circle_poly[F], PlanePoint[F].new(F.1, F.0)) =
        (poly2_eval(circle_poly[F], F.1, F.0) = F.0)
    poly2_eval_circle[F](F.1, F.0)
    poly2_eval(circle_poly[F], F.1, F.0) = F.1 * F.1 + F.0 * F.0 - F.1
    F.1 * F.1 = F.1
    F.0 * F.0 = F.0
    F.1 * F.1 + F.0 * F.0 = F.1
    inverse_left[F](F.1)
    -F.1 + F.1 = F.0
    F.1 + -F.1 = F.0
    F.1 * F.1 + F.0 * F.0 - F.1 = F.0
    poly2_eval(circle_poly[F], F.1, F.0) = F.0
    point_in_zero_set(circle_poly[F], PlanePoint[F].new(F.1, F.0))
}

/// The unit circle contains the point (0, 1).
theorem circle_point_zero_one[F: Field] {
    point_in_zero_set(circle_poly[F], PlanePoint[F].new(F.0, F.1))
} by {
    point_in_zero_set_unfold(circle_poly[F], PlanePoint[F].new(F.0, F.1))
    point_in_zero_set(circle_poly[F], PlanePoint[F].new(F.0, F.1)) =
        (poly2_eval(circle_poly[F], PlanePoint[F].new(F.0, F.1).x,
            PlanePoint[F].new(F.0, F.1).y) = F.0)
    plane_point_new_x(F.0, F.1)
    PlanePoint[F].new(F.0, F.1).x = F.0
    plane_point_new_y(F.0, F.1)
    PlanePoint[F].new(F.0, F.1).y = F.1
    point_in_zero_set(circle_poly[F], PlanePoint[F].new(F.0, F.1)) =
        (poly2_eval(circle_poly[F], F.0, F.1) = F.0)
    poly2_eval_circle[F](F.0, F.1)
    poly2_eval(circle_poly[F], F.0, F.1) = F.0 * F.0 + F.1 * F.1 - F.1
    F.0 * F.0 = F.0
    F.1 * F.1 = F.1
    F.0 * F.0 + F.1 * F.1 = F.1
    inverse_left[F](F.1)
    -F.1 + F.1 = F.0
    F.1 + -F.1 = F.0
    F.0 * F.0 + F.1 * F.1 - F.1 = F.0
    poly2_eval(circle_poly[F], F.0, F.1) = F.0
    point_in_zero_set(circle_poly[F], PlanePoint[F].new(F.0, F.1))
}

/// The unit circle is symmetric: reflecting a point across the diagonal
/// `x = y` preserves membership.
theorem circle_symmetry[F: Field](x: F, y: F) {
    point_in_zero_set(circle_poly[F], PlanePoint[F].new(x, y)) =
        point_in_zero_set(circle_poly[F], PlanePoint[F].new(y, x))
} by {
    point_in_zero_set_unfold(circle_poly[F], PlanePoint[F].new(x, y))
    point_in_zero_set_unfold(circle_poly[F], PlanePoint[F].new(y, x))
    point_in_zero_set(circle_poly[F], PlanePoint[F].new(x, y)) =
        (poly2_eval(circle_poly[F], x, y) = F.0)
    point_in_zero_set(circle_poly[F], PlanePoint[F].new(y, x)) =
        (poly2_eval(circle_poly[F], y, x) = F.0)
    poly2_eval_circle[F](x, y)
    poly2_eval(circle_poly[F], x, y) = x * x + y * y - F.1
    poly2_eval_circle[F](y, x)
    poly2_eval(circle_poly[F], y, x) = y * y + x * x - F.1
    x * x + y * y = y * y + x * x
    x * x + y * y - F.1 = y * y + x * x - F.1
    (poly2_eval(circle_poly[F], x, y) = F.0) = (poly2_eval(circle_poly[F], y, x) = F.0)
    point_in_zero_set(circle_poly[F], PlanePoint[F].new(x, y)) =
        point_in_zero_set(circle_poly[F], PlanePoint[F].new(y, x))
}

// -- (b) the zero set of a product ---------------------------------------------

/// In a field, a product vanishes exactly when a factor vanishes.
theorem field_mul_eq_zero_iff[F: Field](a: F, b: F) {
    (a * b = F.0) = (a = F.0 or b = F.0)
} by {
    if a * b = F.0 {
        field_mul_eq_zero(a, b)
        a = F.0 or b = F.0
        (a * b = F.0) = (a = F.0 or b = F.0)
    }
    if not a * b = F.0 {
        if a = F.0 or b = F.0 {
            field_mul_eq_zero_of_factor(a, b)
            a * b = F.0
            false
        }
        not (a = F.0 or b = F.0)
        (a * b = F.0) = (a = F.0 or b = F.0)
    }
}

/// The zero set of a product is the union of the zero sets of the factors.
theorem zero_set_mul_contains_eq[F: Field](f: Poly2[F], g: Poly2[F], p: PlanePoint[F]) {
    zero_set(poly2_mul(f, g)).contains(p) =
        (zero_set(f).contains(p) or zero_set(g).contains(p))
} by {
    zero_set(poly2_mul(f, g)).contains(p) = point_in_zero_set(poly2_mul(f, g), p)
    zero_set(f).contains(p) = point_in_zero_set(f, p)
    zero_set(g).contains(p) = point_in_zero_set(g, p)
    point_in_zero_set(poly2_mul(f, g), p) = (poly2_eval(poly2_mul(f, g), p.x, p.y) = F.0)
    poly2_eval_mul(f, g, p.x, p.y)
    poly2_eval(poly2_mul(f, g), p.x, p.y) =
        poly2_eval(f, p.x, p.y) * poly2_eval(g, p.x, p.y)
    field_mul_eq_zero_iff(poly2_eval(f, p.x, p.y), poly2_eval(g, p.x, p.y))
    (poly2_eval(f, p.x, p.y) * poly2_eval(g, p.x, p.y) = F.0) =
        (poly2_eval(f, p.x, p.y) = F.0 or poly2_eval(g, p.x, p.y) = F.0)
    point_in_zero_set(poly2_mul(f, g), p) =
        (poly2_eval(f, p.x, p.y) = F.0 or poly2_eval(g, p.x, p.y) = F.0)
    point_in_zero_set(f, p) = (poly2_eval(f, p.x, p.y) = F.0)
    point_in_zero_set(g, p) = (poly2_eval(g, p.x, p.y) = F.0)
    point_in_zero_set(poly2_mul(f, g), p) = (point_in_zero_set(f, p) or point_in_zero_set(g, p))
    zero_set(poly2_mul(f, g)).contains(p) = (point_in_zero_set(f, p) or point_in_zero_set(g, p))
    zero_set(poly2_mul(f, g)).contains(p) =
        (zero_set(f).contains(p) or zero_set(g).contains(p))
}

/// The zero set of a product equals the union of the zero sets of the factors.
theorem zero_set_mul_eq_union[F: Field](f: Poly2[F], g: Poly2[F]) {
    zero_set(poly2_mul(f, g)) = zero_set(f).union(zero_set(g))
} by {
    forall(p: PlanePoint[F]) {
        zero_set_mul_contains_eq(f, g, p)
        zero_set(poly2_mul(f, g)).contains(p) =
            (zero_set(f).contains(p) or zero_set(g).contains(p))
        union_contains_eq(zero_set(f), zero_set(g), p)
        zero_set(f).union(zero_set(g)).contains(p) =
            (zero_set(f).contains(p) or zero_set(g).contains(p))
        zero_set(poly2_mul(f, g)).contains(p) = zero_set(f).union(zero_set(g)).contains(p)
    }
    set_ext(zero_set(poly2_mul(f, g)), zero_set(f).union(zero_set(g)))
}

/// The zero set of `X` is the `y`-axis.
theorem zero_set_x_is_y_axis[R: CommRing](p: PlanePoint[R]) {
    zero_set(biv_x[R]).contains(p) = (p.x = R.0)
} by {
    zero_set(biv_x[R]).contains(p) = point_in_zero_set(biv_x[R], p)
    point_in_zero_set(biv_x[R], p) = (poly2_eval(biv_x[R], p.x, p.y) = R.0)
    poly2_eval_x[R](p.x, p.y)
    poly2_eval(biv_x[R], p.x, p.y) = p.x
    point_in_zero_set(biv_x[R], p) = (p.x = R.0)
    zero_set(biv_x[R]).contains(p) = (p.x = R.0)
}

/// The zero set of `Y` is the `x`-axis.
theorem zero_set_y_is_x_axis[R: CommRing](p: PlanePoint[R]) {
    zero_set(biv_y[R]).contains(p) = (p.y = R.0)
} by {
    zero_set(biv_y[R]).contains(p) = point_in_zero_set(biv_y[R], p)
    point_in_zero_set(biv_y[R], p) = (poly2_eval(biv_y[R], p.x, p.y) = R.0)
    poly2_eval_y[R](p.x, p.y)
    poly2_eval(biv_y[R], p.x, p.y) = p.y
    point_in_zero_set(biv_y[R], p) = (p.y = R.0)
    zero_set(biv_y[R]).contains(p) = (p.y = R.0)
}

/// The zero set of `X * Y` is the union of the two coordinate axes:
/// `V(X * Y) = V(X) ∪ V(Y)`.
theorem zero_set_xy_is_axes[F: Field](p: PlanePoint[F]) {
    zero_set(poly2_mul(biv_x[F], biv_y[F])).contains(p) =
        (zero_set(biv_x[F]).contains(p) or zero_set(biv_y[F]).contains(p))
} by {
    zero_set_mul_contains_eq(biv_x[F], biv_y[F], p)
}

// -- (c) the ideal of a point ---------------------------------------------------

/// The ideal of a point: the bivariate polynomials vanishing at the point.
define ideal_of_point[R: CommRing](p: PlanePoint[R], f: Poly2[R]) -> Bool {
    point_in_zero_set(f, p)
}

/// True if a subset of the bivariate polynomials contains the zero polynomial.
define ideal_of_point_zero_constraint[R: CommRing](p: PlanePoint[R], contains: Poly2[R] -> Bool) -> Bool {
    contains(biv_zero[R])
}

/// True if a subset of the bivariate polynomials is closed under addition.
define ideal_of_point_add_constraint[R: CommRing](p: PlanePoint[R], contains: Poly2[R] -> Bool) -> Bool {
    forall(f: Poly2[R], g: Poly2[R]) {
        contains(f) and contains(g) implies contains(poly2_add(f, g))
    }
}

/// True if a subset of the bivariate polynomials absorbs multiplication by
/// any bivariate polynomial.
define ideal_of_point_absorb_constraint[R: CommRing](p: PlanePoint[R], contains: Poly2[R] -> Bool) -> Bool {
    forall(r: Poly2[R], f: Poly2[R]) {
        contains(f) implies contains(poly2_mul(r, f))
    }
}

/// The three ideal axioms, stated for the ring of bivariate polynomials.
define is_ideal_of_point[R: CommRing](p: PlanePoint[R], contains: Poly2[R] -> Bool) -> Bool {
    ideal_of_point_zero_constraint(p, contains)
    and ideal_of_point_add_constraint(p, contains)
    and ideal_of_point_absorb_constraint(p, contains)
}

/// The ideal of a point is closed under addition.
theorem ideal_of_point_add[R: CommRing](p: PlanePoint[R], f: Poly2[R], g: Poly2[R]) {
    ideal_of_point(p, f) and ideal_of_point(p, g) implies ideal_of_point(p, poly2_add(f, g))
} by {
    if ideal_of_point(p, f) and ideal_of_point(p, g) {
        ideal_of_point(p, f) = point_in_zero_set(f, p)
        point_in_zero_set(f, p)
        point_in_zero_set(f, p) = (poly2_eval(f, p.x, p.y) = R.0)
        poly2_eval(f, p.x, p.y) = R.0
        ideal_of_point(p, g) = point_in_zero_set(g, p)
        point_in_zero_set(g, p)
        point_in_zero_set(g, p) = (poly2_eval(g, p.x, p.y) = R.0)
        poly2_eval(g, p.x, p.y) = R.0
        poly2_eval_add(f, g, p.x, p.y)
        poly2_eval(poly2_add(f, g), p.x, p.y) = poly2_eval(f, p.x, p.y) + poly2_eval(g, p.x, p.y)
        poly2_eval(poly2_add(f, g), p.x, p.y) = R.0 + R.0
        R.0 + R.0 = R.0
        poly2_eval(poly2_add(f, g), p.x, p.y) = R.0
        point_in_zero_set(poly2_add(f, g), p) = (poly2_eval(poly2_add(f, g), p.x, p.y) = R.0)
        point_in_zero_set(poly2_add(f, g), p)
        ideal_of_point(p, poly2_add(f, g)) = point_in_zero_set(poly2_add(f, g), p)
        ideal_of_point(p, poly2_add(f, g))
    }
}

/// The ideal of a point absorbs multiplication by any bivariate polynomial.
theorem ideal_of_point_absorb[R: CommRing](p: PlanePoint[R], r: Poly2[R], f: Poly2[R]) {
    ideal_of_point(p, f) implies ideal_of_point(p, poly2_mul(r, f))
} by {
    if ideal_of_point(p, f) {
        ideal_of_point(p, f) = point_in_zero_set(f, p)
        point_in_zero_set(f, p)
        point_in_zero_set(f, p) = (poly2_eval(f, p.x, p.y) = R.0)
        poly2_eval(f, p.x, p.y) = R.0
        poly2_eval_mul(r, f, p.x, p.y)
        poly2_eval(poly2_mul(r, f), p.x, p.y) = poly2_eval(r, p.x, p.y) * poly2_eval(f, p.x, p.y)
        poly2_eval(poly2_mul(r, f), p.x, p.y) = poly2_eval(r, p.x, p.y) * R.0
        mul_zero_right[R](poly2_eval(r, p.x, p.y))
        poly2_eval(r, p.x, p.y) * R.0 = R.0
        poly2_eval(poly2_mul(r, f), p.x, p.y) = R.0
        point_in_zero_set(poly2_mul(r, f), p) = (poly2_eval(poly2_mul(r, f), p.x, p.y) = R.0)
        point_in_zero_set(poly2_mul(r, f), p)
        ideal_of_point(p, poly2_mul(r, f)) = point_in_zero_set(poly2_mul(r, f), p)
        ideal_of_point(p, poly2_mul(r, f))
    }
}

/// The ideal of a point contains the zero polynomial.
theorem ideal_of_point_zero[R: CommRing](p: PlanePoint[R]) {
    ideal_of_point(p, biv_zero[R])
} by {
    poly2_eval_zero[R](p.x, p.y)
    poly2_eval(biv_zero[R], p.x, p.y) = R.0
    point_in_zero_set(biv_zero[R], p) = (poly2_eval(biv_zero[R], p.x, p.y) = R.0)
    point_in_zero_set(biv_zero[R], p)
    ideal_of_point(p, biv_zero[R]) = point_in_zero_set(biv_zero[R], p)
    ideal_of_point(p, biv_zero[R])
}

/// The ideal of a point is an ideal.
theorem ideal_of_point_is_ideal[R: CommRing](p: PlanePoint[R]) {
    is_ideal_of_point(p, ideal_of_point(p))
} by {
    is_ideal_of_point(p, ideal_of_point(p)) =
        (ideal_of_point_zero_constraint(p, ideal_of_point(p))
        and ideal_of_point_add_constraint(p, ideal_of_point(p))
        and ideal_of_point_absorb_constraint(p, ideal_of_point(p)))
    ideal_of_point_zero_constraint(p, ideal_of_point(p)) = ideal_of_point(p, biv_zero[R])
    ideal_of_point_zero(p)
    ideal_of_point_zero_constraint(p, ideal_of_point(p))
    ideal_of_point_add_constraint(p, ideal_of_point(p)) = forall(f: Poly2[R], g: Poly2[R]) {
        ideal_of_point(p, f) and ideal_of_point(p, g) implies ideal_of_point(p, poly2_add(f, g))
    }
    forall(f: Poly2[R], g: Poly2[R]) {
        ideal_of_point_add(p, f, g)
        ideal_of_point(p, f) and ideal_of_point(p, g) implies ideal_of_point(p, poly2_add(f, g))
    }
    ideal_of_point_add_constraint(p, ideal_of_point(p))
    ideal_of_point_absorb_constraint(p, ideal_of_point(p)) = forall(r: Poly2[R], f: Poly2[R]) {
        ideal_of_point(p, f) implies ideal_of_point(p, poly2_mul(r, f))
    }
    forall(r: Poly2[R], f: Poly2[R]) {
        ideal_of_point_absorb(p, r, f)
        ideal_of_point(p, f) implies ideal_of_point(p, poly2_mul(r, f))
    }
    ideal_of_point_absorb_constraint(p, ideal_of_point(p))
}

// -- (d) the variety of the ideal generated by one polynomial -------------------

/// True if `h` is a multiple of `f` (an element of the ideal generated by `f`).
define in_ideal_generated_by[R: CommRing](f: Poly2[R], h: Poly2[R]) -> Bool {
    exists(g: Poly2[R]) {
        h = poly2_mul(g, f)
    }
}

/// Every polynomial is a multiple of itself: `f` lies in the ideal it generates.
theorem in_ideal_generated_by_self[R: CommRing](f: Poly2[R]) {
    in_ideal_generated_by(f, f)
} by {
    in_ideal_generated_by(f, f) = exists(g: Poly2[R]) {
        f = poly2_mul(g, f)
    }
    poly2_mul_one_left(f)
    poly2_mul(biv_one[R], f) = f
    exists(g: Poly2[R]) {
        g = biv_one[R] and f = poly2_mul(g, f)
    }
    in_ideal_generated_by(f, f)
}

/// True if a point lies in the variety of the ideal generated by `f`: every
/// multiple of `f` vanishes there.
define in_zero_set_of_ideal_generated_by[R: CommRing](f: Poly2[R], p: PlanePoint[R]) -> Bool {
    forall(h: Poly2[R]) {
        in_ideal_generated_by(f, h) implies point_in_zero_set(h, p)
    }
}

/// The variety of the ideal generated by `f` is the zero set of `f`:
/// `V((f)) = V(f)`.
theorem zero_set_of_ideal_generated_by_eq_zero_set[R: CommRing](f: Poly2[R], p: PlanePoint[R]) {
    in_zero_set_of_ideal_generated_by(f, p) = point_in_zero_set(f, p)
} by {
    if in_zero_set_of_ideal_generated_by(f, p) {
        in_zero_set_of_ideal_generated_by(f, p) = forall(h: Poly2[R]) {
            in_ideal_generated_by(f, h) implies point_in_zero_set(h, p)
        }
        in_ideal_generated_by_self(f)
        point_in_zero_set(f, p)
        in_zero_set_of_ideal_generated_by(f, p) = point_in_zero_set(f, p)
    }
    if point_in_zero_set(f, p) {
        point_in_zero_set(f, p) = (poly2_eval(f, p.x, p.y) = R.0)
        poly2_eval(f, p.x, p.y) = R.0
        forall(h: Poly2[R]) {
            if in_ideal_generated_by(f, h) {
                in_ideal_generated_by(f, h) = exists(g: Poly2[R]) {
                    h = poly2_mul(g, f)
                }
                let g: Poly2[R] satisfy {
                    h = poly2_mul(g, f)
                }
                poly2_eval_mul(g, f, p.x, p.y)
                poly2_eval(poly2_mul(g, f), p.x, p.y) = poly2_eval(g, p.x, p.y) * poly2_eval(f, p.x, p.y)
                h = poly2_mul(g, f)
                poly2_eval(h, p.x, p.y) = poly2_eval(poly2_mul(g, f), p.x, p.y)
                poly2_eval(h, p.x, p.y) = poly2_eval(g, p.x, p.y) * poly2_eval(f, p.x, p.y)
                poly2_eval(h, p.x, p.y) = poly2_eval(g, p.x, p.y) * R.0
                mul_zero_right[R](poly2_eval(g, p.x, p.y))
                poly2_eval(g, p.x, p.y) * R.0 = R.0
                poly2_eval(h, p.x, p.y) = R.0
                point_in_zero_set(h, p) = (poly2_eval(h, p.x, p.y) = R.0)
                point_in_zero_set(h, p)
            }
            in_ideal_generated_by(f, h) implies point_in_zero_set(h, p)
        }
        in_zero_set_of_ideal_generated_by(f, p) = forall(h: Poly2[R]) {
            in_ideal_generated_by(f, h) implies point_in_zero_set(h, p)
        }
        in_zero_set_of_ideal_generated_by(f, p)
        in_zero_set_of_ideal_generated_by(f, p) = point_in_zero_set(f, p)
    }
}

/// The unit circle polynomial generates an ideal whose variety is the unit
/// circle: `(1, 0)` lies in `V((X^2 + Y^2 - 1))`.
theorem unit_circle_ideal_contains_one_zero[F: Field] {
    in_zero_set_of_ideal_generated_by(circle_poly[F], PlanePoint[F].new(F.1, F.0))
} by {
    zero_set_of_ideal_generated_by_eq_zero_set(circle_poly[F], PlanePoint[F].new(F.1, F.0))
    in_zero_set_of_ideal_generated_by(circle_poly[F], PlanePoint[F].new(F.1, F.0)) =
        point_in_zero_set(circle_poly[F], PlanePoint[F].new(F.1, F.0))
    circle_point_one_zero[F]
    in_zero_set_of_ideal_generated_by(circle_poly[F], PlanePoint[F].new(F.1, F.0))
}

/// The unit circle polynomial generates an ideal whose variety is the unit
/// circle: `(0, 1)` lies in `V((X^2 + Y^2 - 1))`.
theorem unit_circle_ideal_contains_zero_one[F: Field] {
    in_zero_set_of_ideal_generated_by(circle_poly[F], PlanePoint[F].new(F.0, F.1))
} by {
    zero_set_of_ideal_generated_by_eq_zero_set(circle_poly[F], PlanePoint[F].new(F.0, F.1))
    in_zero_set_of_ideal_generated_by(circle_poly[F], PlanePoint[F].new(F.0, F.1)) =
        point_in_zero_set(circle_poly[F], PlanePoint[F].new(F.0, F.1))
    circle_point_zero_one[F]
    in_zero_set_of_ideal_generated_by(circle_poly[F], PlanePoint[F].new(F.0, F.1))
}

// ---------------------------------------------------------------------------
// (e) The Nullstellensatz
// ---------------------------------------------------------------------------
//
// The Nullstellensatz is the deep converse of the ideal-variety
// correspondence.  Over an algebraically closed field `K`, every polynomial
// vanishing on the variety `V(S)` has a power lying in the ideal generated by
// `S`:
//
//     I(V(S)) = rad((S)),
//
// where `I(V(S))` is the ideal of all polynomials vanishing on `V(S)` and
// `rad((S))` is the radical of the ideal generated by `S`.  This requires
// algebraic closure (roots of every nonconstant univariate polynomial) and
// the full ideal machinery (radicals, Hilbert's basis theorem), neither of
// which this file develops; it is stated here as the target statement of the
// correspondence.  The results above give the easy inclusions for the single
// generator case: `V((f)) = V(f)` and, in a field, `V(f * g) = V(f) ∪ V(g)`.
