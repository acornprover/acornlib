from algebra.semigroup import Semigroup
from data.basic.set import Set, empty_set_is_finite, singleton_contains_eq, singleton_set_is_finite,
    union_contains_eq, union_contains_left, union_contains_right,
    is_subset_monotone_map, is_set_extensive_map, is_set_idempotent_map,
    is_set_closure_operator
from data.basic.functions import predicate_extensionality

// We define subsemigroups with the "bundled" technique, so a subsemigroup carries along
// its semigroup.

/// True if a subset is closed under the semigroup operation.
define subsemigroup_closure_constraint[S: Semigroup](contains: S -> Bool) -> Bool {
    forall(a: S, b: S) {
        contains(a) and contains(b) implies contains(a * b)
    }
}

/// A subsemigroup of a semigroup S, represented as a subset closed under multiplication.
structure Subsemigroup[S: Semigroup] {
    /// True if the given element is a member of this subsemigroup.
    contains: S -> Bool
} constraint {
    subsemigroup_closure_constraint(contains)
}

/// Subsemigroup extensionality: two subsemigroups are equal when they have the same elements.
theorem subsemigroup_ext[S: Semigroup](a: Subsemigroup[S], b: Subsemigroup[S]) {
    (forall(x: S) { a.contains(x) = b.contains(x) }) implies a = b
} by {
    if forall(x: S) { a.contains(x) = b.contains(x) } {
        a.contains = b.contains
    }
}

/// Equal subsemigroups have equal membership predicates.
theorem subsemigroup_eq_contains[S: Semigroup](a: Subsemigroup[S], b: Subsemigroup[S]) {
    a = b implies a.contains = b.contains
}

/// Equal subsemigroups have equal membership at every element.
theorem subsemigroup_eq_contains_at[S: Semigroup](a: Subsemigroup[S], b: Subsemigroup[S], x: S) {
    a = b implies a.contains(x) = b.contains(x)
} by {
    if a = b {
        a.contains = b.contains
        a.contains(x) = b.contains(x)
    }
}

/// Equality of subsemigroups transports predicates on subsemigroups.
theorem subsemigroup_eq_transport_predicate[S: Semigroup](
    p: Subsemigroup[S] -> Bool,
    a: Subsemigroup[S],
    b: Subsemigroup[S]
) {
    a = b and p(a) implies p(b)
}

/// Equality of subsemigroups transports predicates on subsemigroups in the reverse direction.
theorem subsemigroup_eq_transport_predicate_rev[S: Semigroup](
    p: Subsemigroup[S] -> Bool,
    a: Subsemigroup[S],
    b: Subsemigroup[S]
) {
    a = b and p(b) implies p(a)
}

/// Equality of membership predicates determines equality of subsemigroups.
theorem subsemigroup_eq_of_contains_eq[S: Semigroup](a: Subsemigroup[S], b: Subsemigroup[S]) {
    a.contains = b.contains implies a = b
} by {
    if a.contains = b.contains {
        forall(x: S) {
            a.contains(x) = b.contains(x)
        }
        subsemigroup_ext(a, b)
    }
}

/// Pointwise equality of membership determines equality of subsemigroups.
theorem subsemigroup_eq_of_contains_at_eq[S: Semigroup](a: Subsemigroup[S], b: Subsemigroup[S]) {
    (forall(x: S) { a.contains(x) = b.contains(x) }) implies a = b
} by {
    if forall(x: S) { a.contains(x) = b.contains(x) } {
        subsemigroup_ext(a, b)
    }
}

/// The common membership predicate of two subsemigroups.
define subsemigroup_intersection_contains[S: Semigroup](a: Subsemigroup[S], b: Subsemigroup[S],
    x: S) -> Bool {
    a.contains(x) and b.contains(x)
}

/// The common part of two subsemigroups is closed under multiplication.
theorem subsemigroup_intersection_constraint[S: Semigroup](a: Subsemigroup[S], b: Subsemigroup[S]) {
    subsemigroup_closure_constraint(subsemigroup_intersection_contains(a, b))
} by {
    forall(x: S, y: S) {
        if subsemigroup_intersection_contains(a, b, x) and subsemigroup_intersection_contains(a, b, y) {
            a.contains(x)
            b.contains(x)
            a.contains(y)
            b.contains(y)
            subsemigroup_closure_constraint(a.contains)
            subsemigroup_closure_constraint(b.contains)
            subsemigroup_closure_constraint(a.contains) = forall(m: S, n: S) {
                a.contains(m) and a.contains(n) implies a.contains(m * n)
            }
            subsemigroup_closure_constraint(b.contains) = forall(m: S, n: S) {
                b.contains(m) and b.contains(n) implies b.contains(m * n)
            }
            a.contains(x * y)
            b.contains(x * y)
            subsemigroup_intersection_contains(a, b, x * y)
        }
    }
}

let subsemigroup_intersection[S: Semigroup](a: Subsemigroup[S], b: Subsemigroup[S]) -> result: Subsemigroup[S] satisfy {
    Subsemigroup.new(subsemigroup_intersection_contains(a, b)) = Option.some(result)
} by {
    subsemigroup_intersection_constraint(a, b)
}

attributes Subsemigroup[S: Semigroup] {
    /// Subsemigroup extensionality from pointwise equality of membership.
    let ext = subsemigroup_ext[S]

    /// The subset of semigroup elements belonging to this subsemigroup.
    define as_set(self) -> Set[S] {
        Set[S].new(self.contains)
    }

    /// The common part of two subsemigroups.
    let intersection: (Subsemigroup[S], Subsemigroup[S]) -> Subsemigroup[S] = subsemigroup_intersection
}

/// Membership in the underlying set is membership in the subsemigroup.
theorem subsemigroup_as_set_contains_eq[S: Semigroup](s: Subsemigroup[S], x: S) {
    s.as_set.contains(x) = s.contains(x)
} by {
    s.as_set.contains(x) = s.contains(x)
}

/// Membership in a subsemigroup is membership in its underlying set.
theorem subsemigroup_contains_as_set_eq[S: Semigroup](s: Subsemigroup[S], x: S) {
    s.contains(x) = s.as_set.contains(x)
} by {
    subsemigroup_as_set_contains_eq(s, x)
    s.contains(x) = s.as_set.contains(x)
}

/// Equal subsemigroups have equal underlying sets.
theorem subsemigroup_eq_as_set[S: Semigroup](a: Subsemigroup[S], b: Subsemigroup[S]) {
    a = b implies a.as_set = b.as_set
}

/// A subsemigroup is closed under multiplication.
theorem subsemigroup_mul_mem[S: Semigroup](s: Subsemigroup[S], a: S, b: S) {
    s.contains(a) and s.contains(b) implies s.contains(a * b)
} by {
    if s.contains(a) and s.contains(b) {
        subsemigroup_closure_constraint(s.contains)
        subsemigroup_closure_constraint(s.contains) = forall(x: S, y: S) {
            s.contains(x) and s.contains(y) implies s.contains(x * y)
        }
        s.contains(a * b)
    }
}

/// Membership in the intersection means membership in both subsemigroups.
theorem subsemigroup_intersection_contains_eq[S: Semigroup](a: Subsemigroup[S],
    b: Subsemigroup[S], x: S) {
    a.intersection(b).contains(x) = (a.contains(x) and b.contains(x))
} by {
    a.intersection(b).contains(x) = subsemigroup_intersection_contains(a, b, x)
    subsemigroup_intersection_contains(a, b, x) = (a.contains(x) and b.contains(x))
}

/// The intersection is contained in the left subsemigroup.
theorem subsemigroup_intersection_subset_left[S: Semigroup](a: Subsemigroup[S], b: Subsemigroup[S],
    x: S) {
    a.intersection(b).contains(x) implies a.contains(x)
} by {
    if a.intersection(b).contains(x) {
        subsemigroup_intersection_contains_eq(a, b, x)
        a.contains(x)
    }
}

/// The intersection is contained in the right subsemigroup.
theorem subsemigroup_intersection_subset_right[S: Semigroup](a: Subsemigroup[S], b: Subsemigroup[S],
    x: S) {
    a.intersection(b).contains(x) implies b.contains(x)
} by {
    if a.intersection(b).contains(x) {
        subsemigroup_intersection_contains_eq(a, b, x)
        b.contains(x)
    }
}

/// Subsemigroup intersection is commutative.
theorem subsemigroup_intersection_comm[S: Semigroup](a: Subsemigroup[S], b: Subsemigroup[S]) {
    a.intersection(b) = b.intersection(a)
} by {
    forall(x: S) {
        if a.intersection(b).contains(x) {
            subsemigroup_intersection_contains_eq(a, b, x)
            a.contains(x)
            b.contains(x)
            subsemigroup_intersection_contains_eq(b, a, x)
            b.intersection(a).contains(x)
        }
        if b.intersection(a).contains(x) {
            subsemigroup_intersection_contains_eq(b, a, x)
            b.contains(x)
            a.contains(x)
            subsemigroup_intersection_contains_eq(a, b, x)
            a.intersection(b).contains(x)
        }
        a.intersection(b).contains(x) = b.intersection(a).contains(x)
    }
    subsemigroup_ext(a.intersection(b), b.intersection(a))
}

/// Subsemigroup intersection is associative.
theorem subsemigroup_intersection_assoc[S: Semigroup](
    a: Subsemigroup[S],
    b: Subsemigroup[S],
    c: Subsemigroup[S]
) {
    a.intersection(b).intersection(c) = a.intersection(b.intersection(c))
} by {
    let lhs = a.intersection(b).intersection(c)
    let rhs = a.intersection(b.intersection(c))
    forall(x: S) {
        if lhs.contains(x) {
            subsemigroup_intersection_contains_eq(a.intersection(b), c, x)
            a.intersection(b).contains(x)
            c.contains(x)
            subsemigroup_intersection_contains_eq(a, b, x)
            a.contains(x)
            b.contains(x)
            subsemigroup_intersection_contains_eq(b, c, x)
            b.intersection(c).contains(x)
            subsemigroup_intersection_contains_eq(a, b.intersection(c), x)
            rhs.contains(x)
        }
        if rhs.contains(x) {
            subsemigroup_intersection_contains_eq(a, b.intersection(c), x)
            a.contains(x)
            b.intersection(c).contains(x)
            subsemigroup_intersection_contains_eq(b, c, x)
            b.contains(x)
            c.contains(x)
            subsemigroup_intersection_contains_eq(a, b, x)
            a.intersection(b).contains(x)
            subsemigroup_intersection_contains_eq(a.intersection(b), c, x)
            lhs.contains(x)
        }
        lhs.contains(x) = rhs.contains(x)
    }
    subsemigroup_ext(lhs, rhs)
}

/// Subsemigroup intersection is idempotent.
theorem subsemigroup_intersection_idempotent[S: Semigroup](a: Subsemigroup[S]) {
    a.intersection(a) = a
} by {
    forall(x: S) {
        if a.intersection(a).contains(x) {
            subsemigroup_intersection_contains_eq(a, a, x)
            a.contains(x)
        }
        if a.contains(x) {
            subsemigroup_intersection_contains_eq(a, a, x)
            a.intersection(a).contains(x)
        }
        a.intersection(a).contains(x) = a.contains(x)
    }
    subsemigroup_ext(a.intersection(a), a)
}

/// True if every element of one subsemigroup belongs to another.
define subsemigroup_subset[S: Semigroup](a: Subsemigroup[S], b: Subsemigroup[S]) -> Bool {
    forall(x: S) {
        a.contains(x) implies b.contains(x)
    }
}

/// Subsemigroup containment is containment of the underlying sets.
theorem subsemigroup_subset_as_set_eq[S: Semigroup](a: Subsemigroup[S], b: Subsemigroup[S]) {
    subsemigroup_subset(a, b) = a.as_set.subset(b.as_set)
} by {
    if subsemigroup_subset(a, b) {
        forall(x: S) {
            if a.as_set.contains(x) {
                subsemigroup_as_set_contains_eq(a, x)
                a.contains(x)
                b.contains(x)
                subsemigroup_as_set_contains_eq(b, x)
                b.as_set.contains(x)
            }
        }
        a.as_set.subset(b.as_set)
    }
    if a.as_set.subset(b.as_set) {
        forall(x: S) {
            if a.contains(x) {
                subsemigroup_as_set_contains_eq(a, x)
                a.as_set.contains(x)
                a.as_set.subset(b.as_set) = forall(y: S) {
                    a.as_set.contains(y) implies b.as_set.contains(y)
                }
                b.as_set.contains(x)
                subsemigroup_as_set_contains_eq(b, x)
                b.contains(x)
            }
        }
        subsemigroup_subset(a, b)
    }
    subsemigroup_subset(a, b) = a.as_set.subset(b.as_set)
}

/// Subsemigroup containment gives containment of underlying sets.
theorem subsemigroup_as_set_subset_of_subset[S: Semigroup](a: Subsemigroup[S], b: Subsemigroup[S]) {
    subsemigroup_subset(a, b) implies a.as_set.subset(b.as_set)
} by {
    if subsemigroup_subset(a, b) {
        subsemigroup_subset_as_set_eq(a, b)
        a.as_set.subset(b.as_set)
    }
}

/// Containment of underlying sets gives subsemigroup containment.
theorem subsemigroup_subset_of_as_set_subset[S: Semigroup](a: Subsemigroup[S], b: Subsemigroup[S]) {
    a.as_set.subset(b.as_set) implies subsemigroup_subset(a, b)
} by {
    if a.as_set.subset(b.as_set) {
        subsemigroup_subset_as_set_eq(a, b)
        subsemigroup_subset(a, b)
    }
}

/// Subsemigroup inclusion is reflexive.
theorem subsemigroup_subset_refl[S: Semigroup](a: Subsemigroup[S]) {
    subsemigroup_subset(a, a)
} by {
    forall(x: S) {
        a.contains(x) implies a.contains(x)
    }
}

/// Subsemigroup inclusion is transitive.
theorem subsemigroup_subset_trans[S: Semigroup](
    a: Subsemigroup[S],
    b: Subsemigroup[S],
    c: Subsemigroup[S]
) {
    subsemigroup_subset(a, b) and subsemigroup_subset(b, c) implies subsemigroup_subset(a, c)
} by {
    if subsemigroup_subset(a, b) and subsemigroup_subset(b, c) {
        forall(x: S) {
            if a.contains(x) {
                b.contains(x)
                c.contains(x)
            }
        }
    }
}

/// The intersection of two subsemigroups is contained in the left subsemigroup.
theorem subsemigroup_intersection_subset_left_relation[S: Semigroup](
    a: Subsemigroup[S],
    b: Subsemigroup[S]
) {
    subsemigroup_subset(a.intersection(b), a)
} by {
    forall(x: S) {
        if a.intersection(b).contains(x) {
            subsemigroup_intersection_contains_eq(a, b, x)
            a.contains(x)
        }
    }
}

/// The intersection of two subsemigroups is contained in the right subsemigroup.
theorem subsemigroup_intersection_subset_right_relation[S: Semigroup](
    a: Subsemigroup[S],
    b: Subsemigroup[S]
) {
    subsemigroup_subset(a.intersection(b), b)
} by {
    forall(x: S) {
        if a.intersection(b).contains(x) {
            subsemigroup_intersection_contains_eq(a, b, x)
            b.contains(x)
        }
    }
}

/// Every common subsemigroup of two subsemigroups is contained in their intersection.
theorem subsemigroup_subset_intersection_of_subset_left_right[S: Semigroup](
    c: Subsemigroup[S],
    a: Subsemigroup[S],
    b: Subsemigroup[S]
) {
    subsemigroup_subset(c, a) and subsemigroup_subset(c, b) implies subsemigroup_subset(c, a.intersection(b))
} by {
    if subsemigroup_subset(c, a) and subsemigroup_subset(c, b) {
        forall(x: S) {
            if c.contains(x) {
                a.contains(x)
                b.contains(x)
                subsemigroup_intersection_contains_eq(a, b, x)
                a.intersection(b).contains(x)
            }
        }
    }
}

/// Containment in an intersection is equivalent to containment in both subsemigroups.
theorem subsemigroup_subset_intersection_iff[S: Semigroup](
    c: Subsemigroup[S],
    a: Subsemigroup[S],
    b: Subsemigroup[S]
) {
    subsemigroup_subset(c, a.intersection(b)) = (subsemigroup_subset(c, a) and subsemigroup_subset(c, b))
} by {
    if subsemigroup_subset(c, a.intersection(b)) {
        subsemigroup_intersection_subset_left_relation(a, b)
        subsemigroup_subset_trans(c, a.intersection(b), a)
        subsemigroup_subset(c, a)
        subsemigroup_intersection_subset_right_relation(a, b)
        subsemigroup_subset_trans(c, a.intersection(b), b)
        subsemigroup_subset(c, b)
        subsemigroup_subset(c, a) and subsemigroup_subset(c, b)
    }
    if subsemigroup_subset(c, a) and subsemigroup_subset(c, b) {
        subsemigroup_subset_intersection_of_subset_left_right(c, a, b)
        subsemigroup_subset(c, a.intersection(b))
    }
    subsemigroup_subset(c, a.intersection(b)) = (subsemigroup_subset(c, a) and subsemigroup_subset(c, b))
}

/// The empty subset of any semigroup is closed under multiplication.
define empty_contains[S: Semigroup](a: S) -> Bool {
    false
}

theorem empty_subsemigroup_constraint[S: Semigroup] {
    subsemigroup_closure_constraint(empty_contains[S])
} by {
    forall(a: S, b: S) {
        if empty_contains[S](a) and empty_contains[S](b) {
            false
        }
    }
}

/// The empty subsemigroup, containing no elements.
let empty_subsemigroup[S: Semigroup]: Subsemigroup[S] satisfy {
    Subsemigroup.new(empty_contains[S]) = Option.some(empty_subsemigroup)
}

theorem empty_subsemigroup_contains_nothing[S: Semigroup](s: S) {
    not empty_subsemigroup[S].contains(s)
}

/// Membership in the empty subsemigroup is false.
theorem empty_subsemigroup_contains_eq[S: Semigroup](s: S) {
    empty_subsemigroup[S].contains(s) = false
} by {
    empty_subsemigroup_contains_nothing(s)
}

/// The empty subsemigroup is contained in every subsemigroup.
theorem empty_subsemigroup_subset[S: Semigroup](s: Subsemigroup[S]) {
    subsemigroup_subset(empty_subsemigroup[S], s)
} by {
    forall(x: S) {
        if empty_subsemigroup[S].contains(x) {
            empty_subsemigroup_contains_nothing(x)
            false
        }
    }
}

/// The full subset of any semigroup is closed under multiplication.
define full_contains[S: Semigroup](a: S) -> Bool {
    true
}

theorem full_subsemigroup_constraint[S: Semigroup] {
    subsemigroup_closure_constraint(full_contains[S])
}

/// The full subsemigroup, containing every element of the semigroup.
let full_subsemigroup[S: Semigroup]: Subsemigroup[S] satisfy {
    Subsemigroup.new(full_contains[S]) = Option.some(full_subsemigroup)
}

theorem full_subsemigroup_contains_everything[S: Semigroup](s: S) {
    full_subsemigroup[S].contains(s)
}

/// Membership in the full subsemigroup is always true.
theorem full_subsemigroup_contains_eq[S: Semigroup](s: S) {
    full_subsemigroup[S].contains(s) = true
} by {
    full_subsemigroup_contains_everything(s)
}

/// Every subsemigroup is contained in the full subsemigroup.
theorem subsemigroup_subset_full[S: Semigroup](s: Subsemigroup[S]) {
    subsemigroup_subset(s, full_subsemigroup[S])
} by {
    forall(x: S) {
        if s.contains(x) {
            full_subsemigroup_contains_everything(x)
        }
    }
}

/// Mutual inclusion of subsemigroups forces equality.
theorem subsemigroup_subset_antisymm[S: Semigroup](a: Subsemigroup[S], b: Subsemigroup[S]) {
    subsemigroup_subset(a, b) and subsemigroup_subset(b, a) implies a = b
} by {
    if subsemigroup_subset(a, b) and subsemigroup_subset(b, a) {
        subsemigroup_subset(a, b) = forall(y: S) {
            a.contains(y) implies b.contains(y)
        }
        subsemigroup_subset(b, a) = forall(y: S) {
            b.contains(y) implies a.contains(y)
        }
        forall(x: S) {
            if a.contains(x) {
                b.contains(x)
            }
            if b.contains(x) {
                a.contains(x)
            }
            a.contains(x) = b.contains(x)
        }
        predicate_extensionality(a.contains, b.contains)
        a.contains = b.contains
        a = b
    }
}

/// Equality of subsemigroups is equivalent to mutual inclusion.
theorem subsemigroup_eq_iff_subset_both[S: Semigroup](a: Subsemigroup[S], b: Subsemigroup[S]) {
    a = b = (subsemigroup_subset(a, b) and subsemigroup_subset(b, a))
} by {
    if a = b {
        subsemigroup_subset_refl(a)
        subsemigroup_subset(a, b)
        subsemigroup_subset(b, a)
        subsemigroup_subset(a, b) and subsemigroup_subset(b, a)
    }
    if subsemigroup_subset(a, b) and subsemigroup_subset(b, a) {
        subsemigroup_subset_antisymm(a, b)
        a = b
    }
    a = b = (subsemigroup_subset(a, b) and subsemigroup_subset(b, a))
}

/// Intersecting with a larger subsemigroup gives the smaller subsemigroup.
theorem subsemigroup_intersection_eq_left_of_subset[S: Semigroup](
    a: Subsemigroup[S],
    b: Subsemigroup[S]
) {
    subsemigroup_subset(a, b) implies a.intersection(b) = a
} by {
    if subsemigroup_subset(a, b) {
        subsemigroup_intersection_subset_left_relation(a, b)
        subsemigroup_subset_intersection_of_subset_left_right(a, a, b)
        subsemigroup_subset(a, a.intersection(b))
        subsemigroup_subset_antisymm(a.intersection(b), a)
        a.intersection(b) = a
    }
}

/// Intersecting with a larger subsemigroup on the left gives the smaller subsemigroup.
theorem subsemigroup_intersection_eq_right_of_subset[S: Semigroup](
    a: Subsemigroup[S],
    b: Subsemigroup[S]
) {
    subsemigroup_subset(b, a) implies a.intersection(b) = b
} by {
    if subsemigroup_subset(b, a) {
        subsemigroup_intersection_subset_right_relation(a, b)
        subsemigroup_subset_intersection_of_subset_left_right(b, a, b)
        subsemigroup_subset(b, a.intersection(b))
        subsemigroup_subset_antisymm(a.intersection(b), b)
        a.intersection(b) = b
    }
}

/// Intersecting the empty subsemigroup on the left gives the empty subsemigroup.
theorem empty_subsemigroup_intersection_left[S: Semigroup](s: Subsemigroup[S]) {
    empty_subsemigroup[S].intersection(s) = empty_subsemigroup[S]
} by {
    empty_subsemigroup_subset(s)
    subsemigroup_intersection_eq_left_of_subset(empty_subsemigroup[S], s)
}

/// Intersecting the empty subsemigroup on the right gives the empty subsemigroup.
theorem empty_subsemigroup_intersection_right[S: Semigroup](s: Subsemigroup[S]) {
    s.intersection(empty_subsemigroup[S]) = empty_subsemigroup[S]
} by {
    empty_subsemigroup_subset(s)
    subsemigroup_intersection_eq_right_of_subset(s, empty_subsemigroup[S])
}

/// Intersecting the full subsemigroup on the left gives the other subsemigroup.
theorem full_subsemigroup_intersection_left[S: Semigroup](s: Subsemigroup[S]) {
    full_subsemigroup[S].intersection(s) = s
} by {
    subsemigroup_subset_full(s)
    subsemigroup_intersection_eq_right_of_subset(full_subsemigroup[S], s)
}

/// Intersecting the full subsemigroup on the right gives the other subsemigroup.
theorem full_subsemigroup_intersection_right[S: Semigroup](s: Subsemigroup[S]) {
    s.intersection(full_subsemigroup[S]) = s
} by {
    subsemigroup_subset_full(s)
    subsemigroup_intersection_eq_left_of_subset(s, full_subsemigroup[S])
}

/// True if every element of a set belongs to a subsemigroup.
define set_subset_subsemigroup[S: Semigroup](a: Set[S], s: Subsemigroup[S]) -> Bool {
    forall(x: S) {
        a.contains(x) implies s.contains(x)
    }
}

/// Set containment in a subsemigroup is containment in its underlying set.
theorem set_subset_subsemigroup_as_set_eq[S: Semigroup](a: Set[S], s: Subsemigroup[S]) {
    set_subset_subsemigroup(a, s) = a.subset(s.as_set)
} by {
    if set_subset_subsemigroup(a, s) {
        forall(x: S) {
            if a.contains(x) {
                s.contains(x)
                subsemigroup_as_set_contains_eq(s, x)
                s.as_set.contains(x)
            }
        }
        a.subset(s.as_set)
    }
    if a.subset(s.as_set) {
        forall(x: S) {
            if a.contains(x) {
                a.subset(s.as_set) = forall(y: S) {
                    a.contains(y) implies s.as_set.contains(y)
                }
                s.as_set.contains(x)
                subsemigroup_as_set_contains_eq(s, x)
                s.contains(x)
            }
        }
        set_subset_subsemigroup(a, s)
    }
    set_subset_subsemigroup(a, s) = a.subset(s.as_set)
}

/// Set containment in a subsemigroup gives containment in its underlying set.
theorem set_as_set_subset_of_subset_subsemigroup[S: Semigroup](a: Set[S], s: Subsemigroup[S]) {
    set_subset_subsemigroup(a, s) implies a.subset(s.as_set)
} by {
    if set_subset_subsemigroup(a, s) {
        set_subset_subsemigroup_as_set_eq(a, s)
        a.subset(s.as_set)
    }
}

/// Containment in the underlying set gives set containment in the subsemigroup.
theorem set_subset_subsemigroup_of_as_set_subset[S: Semigroup](a: Set[S], s: Subsemigroup[S]) {
    a.subset(s.as_set) implies set_subset_subsemigroup(a, s)
} by {
    if a.subset(s.as_set) {
        set_subset_subsemigroup_as_set_eq(a, s)
        set_subset_subsemigroup(a, s)
    }
}

/// The underlying set of a subsemigroup is contained in the subsemigroup.
theorem subsemigroup_as_set_subset_self[S: Semigroup](s: Subsemigroup[S]) {
    set_subset_subsemigroup(s.as_set, s)
} by {
    forall(x: S) {
        if s.as_set.contains(x) {
            s.contains(x)
        }
    }
}

/// Set containment is preserved by enlarging the target subsemigroup.
theorem set_subset_subsemigroup_trans[S: Semigroup](
    a: Set[S],
    s: Subsemigroup[S],
    t: Subsemigroup[S]
) {
    set_subset_subsemigroup(a, s) and subsemigroup_subset(s, t) implies set_subset_subsemigroup(a, t)
} by {
    if set_subset_subsemigroup(a, s) and subsemigroup_subset(s, t) {
        forall(x: S) {
            if a.contains(x) {
                s.contains(x)
                t.contains(x)
            }
        }
    }
}

/// Set containment in a subsemigroup is preserved by enlarging the set.
theorem set_subset_subsemigroup_of_set_subset[S: Semigroup](
    a: Set[S],
    b: Set[S],
    s: Subsemigroup[S]
) {
    a.subset(b) and set_subset_subsemigroup(b, s) implies set_subset_subsemigroup(a, s)
} by {
    if a.subset(b) and set_subset_subsemigroup(b, s) {
        forall(x: S) {
            if a.contains(x) {
                a.subset(b) = forall(y: S) {
                    a.contains(y) implies b.contains(y)
                }
                b.contains(x)
                s.contains(x)
            }
        }
    }
}

/// True if an element belongs to every subsemigroup that contains a given set.
define subsemigroup_closure_contains[S: Semigroup](a: Set[S], x: S) -> Bool {
    forall(s: Subsemigroup[S]) {
        set_subset_subsemigroup(a, s) implies s.contains(x)
    }
}

/// Membership in the closure gives membership in each subsemigroup that contains the set.
theorem subsemigroup_closure_contains_of_set_subset_raw[S: Semigroup](
    a: Set[S],
    s: Subsemigroup[S],
    x: S
) {
    subsemigroup_closure_contains(a, x) and set_subset_subsemigroup(a, s) implies s.contains(x)
} by {
    if subsemigroup_closure_contains(a, x) and set_subset_subsemigroup(a, s) {
        subsemigroup_closure_contains(a, x) = forall(t: Subsemigroup[S]) {
            set_subset_subsemigroup(a, t) implies t.contains(x)
        }
        s.contains(x)
    }
}

/// The subsemigroup closure of a set is closed under multiplication.
theorem subsemigroup_closure_closure_constraint[S: Semigroup](a: Set[S]) {
    subsemigroup_closure_constraint(subsemigroup_closure_contains(a))
} by {
    forall(x: S, y: S) {
        if subsemigroup_closure_contains(a, x) and subsemigroup_closure_contains(a, y) {
            forall(s: Subsemigroup[S]) {
                if set_subset_subsemigroup(a, s) {
                    subsemigroup_closure_contains_of_set_subset_raw(a, s, x)
                    s.contains(x)
                    subsemigroup_closure_contains_of_set_subset_raw(a, s, y)
                    s.contains(y)
                    subsemigroup_mul_mem(s, x, y)
                    s.contains(x * y)
                }
            }
            subsemigroup_closure_contains(a, x * y)
        }
    }
}

/// The subsemigroup generated by a set satisfies the subsemigroup law.
theorem subsemigroup_generated_constraint[S: Semigroup](a: Set[S]) {
    subsemigroup_closure_constraint(subsemigroup_closure_contains(a))
} by {
    subsemigroup_closure_closure_constraint(a)
}

/// The smallest subsemigroup containing a set.
let subsemigroup_closure[S: Semigroup](a: Set[S]) -> result: Subsemigroup[S] satisfy {
    Subsemigroup.new(subsemigroup_closure_contains(a)) = Option.some(result)
} by {
    subsemigroup_generated_constraint(a)
}

/// Membership in the closure means membership in every subsemigroup containing the set.
theorem subsemigroup_closure_contains_eq[S: Semigroup](a: Set[S], x: S) {
    subsemigroup_closure(a).contains(x) = subsemigroup_closure_contains(a, x)
} by {
    subsemigroup_closure(a).contains(x) = subsemigroup_closure_contains(a, x)
}

/// Every generator belongs to the subsemigroup closure.
theorem subsemigroup_subset_closure[S: Semigroup](a: Set[S]) {
    set_subset_subsemigroup(a, subsemigroup_closure(a))
} by {
    forall(x: S) {
        if a.contains(x) {
            forall(s: Subsemigroup[S]) {
                if set_subset_subsemigroup(a, s) {
                    s.contains(x)
                }
            }
            subsemigroup_closure_contains(a, x)
            subsemigroup_closure_contains_eq(a, x)
            subsemigroup_closure(a).contains(x)
        }
    }
}

/// A generator is a member of the subsemigroup closure.
theorem subsemigroup_closure_contains_of_set_contains[S: Semigroup](a: Set[S], x: S) {
    a.contains(x) implies subsemigroup_closure(a).contains(x)
} by {
    if a.contains(x) {
        subsemigroup_subset_closure(a)
        subsemigroup_closure(a).contains(x)
    }
}

/// The subsemigroup closure is contained in any subsemigroup containing the set.
theorem subsemigroup_closure_subset_of_set_subset[S: Semigroup](a: Set[S], s: Subsemigroup[S]) {
    set_subset_subsemigroup(a, s) implies subsemigroup_subset(subsemigroup_closure(a), s)
} by {
    if set_subset_subsemigroup(a, s) {
        forall(x: S) {
            if subsemigroup_closure(a).contains(x) {
                subsemigroup_closure_contains_eq(a, x)
                subsemigroup_closure_contains(a, x)
                subsemigroup_closure_contains_of_set_subset_raw(a, s, x)
            }
        }
    }
}

/// The subsemigroup closure is the least subsemigroup containing the set.
theorem subsemigroup_closure_le_iff_set_subset[S: Semigroup](a: Set[S], s: Subsemigroup[S]) {
    subsemigroup_subset(subsemigroup_closure(a), s) = set_subset_subsemigroup(a, s)
} by {
    if subsemigroup_subset(subsemigroup_closure(a), s) {
        subsemigroup_subset_closure(a)
        set_subset_subsemigroup_trans(a, subsemigroup_closure(a), s)
        set_subset_subsemigroup(a, s)
    }
    if set_subset_subsemigroup(a, s) {
        subsemigroup_closure_subset_of_set_subset(a, s)
        subsemigroup_subset(subsemigroup_closure(a), s)
    }
    subsemigroup_subset(subsemigroup_closure(a), s) = set_subset_subsemigroup(a, s)
}

/// The subsemigroup closure and underlying set maps form a set-containment adjunction.
theorem subsemigroup_closure_as_set_galois_connection[S: Semigroup](a: Set[S], s: Subsemigroup[S]) {
    subsemigroup_closure(a).as_set.subset(s.as_set) = a.subset(s.as_set)
} by {
    subsemigroup_subset_as_set_eq(subsemigroup_closure(a), s)
    subsemigroup_closure_le_iff_set_subset(a, s)
    set_subset_subsemigroup_as_set_eq(a, s)
    subsemigroup_closure(a).as_set.subset(s.as_set) = a.subset(s.as_set)
}

/// The closure of the underlying set of a subsemigroup is the subsemigroup itself.
theorem subsemigroup_closure_as_set[S: Semigroup](s: Subsemigroup[S]) {
    subsemigroup_closure(s.as_set) = s
} by {
    subsemigroup_as_set_subset_self(s)
    subsemigroup_closure_subset_of_set_subset(s.as_set, s)
    subsemigroup_subset(subsemigroup_closure(s.as_set), s)
    subsemigroup_subset_closure(s.as_set)
    forall(x: S) {
        if s.contains(x) {
            s.as_set.contains(x)
            subsemigroup_closure(s.as_set).contains(x)
        }
    }
    subsemigroup_subset(s, subsemigroup_closure(s.as_set))
    subsemigroup_subset_antisymm(subsemigroup_closure(s.as_set), s)
}

/// Subsemigroup closure is monotone with respect to set inclusion.
theorem subsemigroup_closure_mono[S: Semigroup](a: Set[S], b: Set[S]) {
    a.subset(b) implies subsemigroup_subset(subsemigroup_closure(a), subsemigroup_closure(b))
} by {
    if a.subset(b) {
        subsemigroup_subset_closure(b)
        set_subset_subsemigroup_of_set_subset(a, b, subsemigroup_closure(b))
        set_subset_subsemigroup(a, subsemigroup_closure(b))
        subsemigroup_closure_subset_of_set_subset(a, subsemigroup_closure(b))
        subsemigroup_subset(subsemigroup_closure(a), subsemigroup_closure(b))
    }
}

/// Equal sets have equal subsemigroup closures.
theorem subsemigroup_closure_eq_of_set_eq[S: Semigroup](a: Set[S], b: Set[S]) {
    a = b implies subsemigroup_closure(a) = subsemigroup_closure(b)
} by {
    if a = b {
        subsemigroup_closure(a) = subsemigroup_closure(b)
    }
}

/// Applying subsemigroup closure twice gives the same subsemigroup.
theorem subsemigroup_closure_idempotent[S: Semigroup](a: Set[S]) {
    subsemigroup_closure(subsemigroup_closure(a).as_set) = subsemigroup_closure(a)
} by {
    subsemigroup_closure_as_set(subsemigroup_closure(a))
}

/// The set closure induced by subsemigroup generation.
define subsemigroup_set_closure[S: Semigroup](a: Set[S]) -> Set[S] {
    subsemigroup_closure(a).as_set
}

/// The subsemigroup set closure is the underlying set of the generated subsemigroup.
theorem subsemigroup_set_closure_at[S: Semigroup](a: Set[S]) {
    subsemigroup_set_closure(a) = subsemigroup_closure(a).as_set
}

/// The subsemigroup set closure contains the original set.
theorem subsemigroup_set_closure_extensive[S: Semigroup](a: Set[S]) {
    a.subset(subsemigroup_set_closure(a))
} by {
    subsemigroup_set_closure_at(a)
    subsemigroup_subset_closure(a)
    set_subset_subsemigroup_as_set_eq(a, subsemigroup_closure(a))
    a.subset(subsemigroup_closure(a).as_set)
    a.subset(subsemigroup_set_closure(a))
}

/// The subsemigroup set closure preserves inclusion.
theorem subsemigroup_set_closure_mono[S: Semigroup](a: Set[S], b: Set[S]) {
    a.subset(b) implies subsemigroup_set_closure(a).subset(subsemigroup_set_closure(b))
} by {
    if a.subset(b) {
        subsemigroup_closure_mono(a, b)
        subsemigroup_subset(subsemigroup_closure(a), subsemigroup_closure(b))
        subsemigroup_subset_as_set_eq(subsemigroup_closure(a), subsemigroup_closure(b))
        subsemigroup_closure(a).as_set.subset(subsemigroup_closure(b).as_set)
        subsemigroup_set_closure_at(a)
        subsemigroup_set_closure_at(b)
        subsemigroup_set_closure(a).subset(subsemigroup_set_closure(b))
    }
}

/// The subsemigroup set closure is unchanged after two applications.
theorem subsemigroup_set_closure_idempotent[S: Semigroup](a: Set[S]) {
    subsemigroup_set_closure(subsemigroup_set_closure(a)) = subsemigroup_set_closure(a)
} by {
    subsemigroup_set_closure_at(a)
    subsemigroup_set_closure_at(subsemigroup_set_closure(a))
    subsemigroup_closure_idempotent(a)
    subsemigroup_set_closure(subsemigroup_set_closure(a)) = subsemigroup_set_closure(a)
}

/// Subsemigroup set closure is monotone as a set map.
theorem subsemigroup_set_closure_is_monotone[S: Semigroup] {
    is_subset_monotone_map(subsemigroup_set_closure[S])
} by {
    forall(a: Set[S], b: Set[S]) {
        if a.subset(b) {
            subsemigroup_set_closure_mono(a, b)
            subsemigroup_set_closure(a).subset(subsemigroup_set_closure(b))
        }
    }
}

/// Subsemigroup set closure is extensive as a set map.
theorem subsemigroup_set_closure_is_extensive[S: Semigroup] {
    is_set_extensive_map(subsemigroup_set_closure[S])
} by {
    forall(a: Set[S]) {
        subsemigroup_set_closure_extensive(a)
        a.subset(subsemigroup_set_closure(a))
    }
}

/// Subsemigroup set closure is idempotent as a set map.
theorem subsemigroup_set_closure_is_idempotent[S: Semigroup] {
    is_set_idempotent_map(subsemigroup_set_closure[S])
} by {
    forall(a: Set[S]) {
        subsemigroup_set_closure_idempotent(a)
        subsemigroup_set_closure(subsemigroup_set_closure(a)) = subsemigroup_set_closure(a)
    }
}

/// Subsemigroup generation induces a closure operator on sets.
theorem subsemigroup_set_closure_operator[S: Semigroup] {
    is_set_closure_operator(subsemigroup_set_closure[S])
} by {
    subsemigroup_set_closure_is_monotone[S]
    subsemigroup_set_closure_is_extensive[S]
    subsemigroup_set_closure_is_idempotent[S]
    is_set_closure_operator(subsemigroup_set_closure[S])
}

/// The closure of the empty set is the empty subsemigroup.
theorem subsemigroup_closure_empty[S: Semigroup] {
    subsemigroup_closure(Set[S].empty_set) = empty_subsemigroup[S]
} by {
    set_subset_subsemigroup(Set[S].empty_set, empty_subsemigroup[S])
    subsemigroup_closure_subset_of_set_subset(Set[S].empty_set, empty_subsemigroup[S])
    subsemigroup_subset(subsemigroup_closure(Set[S].empty_set), empty_subsemigroup[S])
    empty_subsemigroup_subset(subsemigroup_closure(Set[S].empty_set))
    subsemigroup_subset_antisymm(subsemigroup_closure(Set[S].empty_set), empty_subsemigroup[S])
}

/// The closure of the universal set is the full subsemigroup.
theorem subsemigroup_closure_universal[S: Semigroup] {
    subsemigroup_closure(Set[S].universal_set) = full_subsemigroup[S]
} by {
    subsemigroup_subset_full(subsemigroup_closure(Set[S].universal_set))
    forall(x: S) {
        if full_subsemigroup[S].contains(x) {
            Set[S].universal_set.contains(x)
            subsemigroup_closure_contains_of_set_contains(Set[S].universal_set, x)
            subsemigroup_closure(Set[S].universal_set).contains(x)
        }
    }
    subsemigroup_subset(full_subsemigroup[S], subsemigroup_closure(Set[S].universal_set))
    subsemigroup_subset_antisymm(subsemigroup_closure(Set[S].universal_set), full_subsemigroup[S])
}

/// The least subsemigroup containing two subsemigroups.
define subsemigroup_sup[S: Semigroup](a: Subsemigroup[S], b: Subsemigroup[S]) -> Subsemigroup[S] {
    subsemigroup_closure(a.as_set.union(b.as_set))
}

/// The join of two subsemigroups is the closure of the union of their underlying sets.
theorem subsemigroup_sup_eq_closure_union[S: Semigroup](a: Subsemigroup[S], b: Subsemigroup[S]) {
    subsemigroup_sup(a, b) = subsemigroup_closure(a.as_set.union(b.as_set))
}

/// The left subsemigroup is contained in the join.
theorem subsemigroup_subset_sup_left[S: Semigroup](a: Subsemigroup[S], b: Subsemigroup[S]) {
    subsemigroup_subset(a, subsemigroup_sup(a, b))
} by {
    forall(x: S) {
        if a.contains(x) {
            subsemigroup_as_set_contains_eq(a, x)
            a.as_set.contains(x)
            union_contains_left(a.as_set, b.as_set, x)
            a.as_set.union(b.as_set).contains(x)
            subsemigroup_closure_contains_of_set_contains(a.as_set.union(b.as_set), x)
            subsemigroup_closure(a.as_set.union(b.as_set)).contains(x)
            subsemigroup_sup(a, b).contains(x)
        }
    }
}

/// The right subsemigroup is contained in the join.
theorem subsemigroup_subset_sup_right[S: Semigroup](a: Subsemigroup[S], b: Subsemigroup[S]) {
    subsemigroup_subset(b, subsemigroup_sup(a, b))
} by {
    forall(x: S) {
        if b.contains(x) {
            subsemigroup_as_set_contains_eq(b, x)
            b.as_set.contains(x)
            union_contains_right(a.as_set, b.as_set, x)
            a.as_set.union(b.as_set).contains(x)
            subsemigroup_closure_contains_of_set_contains(a.as_set.union(b.as_set), x)
            subsemigroup_closure(a.as_set.union(b.as_set)).contains(x)
            subsemigroup_sup(a, b).contains(x)
        }
    }
}

/// The join is contained in every common upper bound.
theorem subsemigroup_sup_subset_of_subset_left_right[S: Semigroup](
    a: Subsemigroup[S],
    b: Subsemigroup[S],
    c: Subsemigroup[S]
) {
    subsemigroup_subset(a, c) and subsemigroup_subset(b, c) implies subsemigroup_subset(subsemigroup_sup(a, b), c)
} by {
    if subsemigroup_subset(a, c) and subsemigroup_subset(b, c) {
        forall(x: S) {
            if a.as_set.union(b.as_set).contains(x) {
                union_contains_eq(a.as_set, b.as_set, x)
                if a.as_set.contains(x) {
                    subsemigroup_as_set_contains_eq(a, x)
                    a.contains(x)
                    c.contains(x)
                } else {
                    b.as_set.contains(x)
                    subsemigroup_as_set_contains_eq(b, x)
                    b.contains(x)
                    c.contains(x)
                }
            }
        }
        set_subset_subsemigroup(a.as_set.union(b.as_set), c)
        subsemigroup_closure_subset_of_set_subset(a.as_set.union(b.as_set), c)
        subsemigroup_subset(subsemigroup_closure(a.as_set.union(b.as_set)), c)
        subsemigroup_subset(subsemigroup_sup(a, b), c)
    }
}

/// Containment of a join is equivalent to containment of both subsemigroups.
theorem subsemigroup_sup_subset_iff[S: Semigroup](
    a: Subsemigroup[S],
    b: Subsemigroup[S],
    c: Subsemigroup[S]
) {
    subsemigroup_subset(subsemigroup_sup(a, b), c) = (subsemigroup_subset(a, c) and subsemigroup_subset(b, c))
} by {
    if subsemigroup_subset(subsemigroup_sup(a, b), c) {
        subsemigroup_subset_sup_left(a, b)
        subsemigroup_subset_trans(a, subsemigroup_sup(a, b), c)
        subsemigroup_subset(a, c)
        subsemigroup_subset_sup_right(a, b)
        subsemigroup_subset_trans(b, subsemigroup_sup(a, b), c)
        subsemigroup_subset(b, c)
        subsemigroup_subset(a, c) and subsemigroup_subset(b, c)
    }
    if subsemigroup_subset(a, c) and subsemigroup_subset(b, c) {
        subsemigroup_sup_subset_of_subset_left_right(a, b, c)
        subsemigroup_subset(subsemigroup_sup(a, b), c)
    }
    subsemigroup_subset(subsemigroup_sup(a, b), c) = (subsemigroup_subset(a, c) and subsemigroup_subset(b, c))
}

attributes Subsemigroup[S: Semigroup] {
    /// The smallest subsemigroup containing a set.
    let closure: Set[S] -> Subsemigroup[S] = subsemigroup_closure

    /// The least subsemigroup containing this subsemigroup and another subsemigroup.
    let sup: (Subsemigroup[S], Subsemigroup[S]) -> Subsemigroup[S] = subsemigroup_sup
}

/// True if a subsemigroup is generated by a finite set.
define subsemigroup_is_finitely_generated[S: Semigroup](s: Subsemigroup[S]) -> Bool {
    exists(a: Set[S]) {
        a.is_finite and subsemigroup_closure(a) = s
    }
}

/// A finitely generated subsemigroup has a finite generating set.
theorem subsemigroup_is_finitely_generated_witness[S: Semigroup](s: Subsemigroup[S]) {
    subsemigroup_is_finitely_generated(s) implies exists(a: Set[S]) {
        a.is_finite and subsemigroup_closure(a) = s
    }
}

/// A subsemigroup equal to the closure of a finite set is finitely generated.
theorem subsemigroup_is_finitely_generated_of_closure_eq[S: Semigroup](
    a: Set[S],
    s: Subsemigroup[S]
) {
    a.is_finite and subsemigroup_closure(a) = s implies subsemigroup_is_finitely_generated(s)
} by {
    if a.is_finite and subsemigroup_closure(a) = s {
        exists(b: Set[S]) {
            b.is_finite and subsemigroup_closure(b) = s
        }
        subsemigroup_is_finitely_generated(s)
    }
}

/// Equality preserves finite generation of subsemigroups.
theorem subsemigroup_is_finitely_generated_of_eq[S: Semigroup](
    s: Subsemigroup[S],
    t: Subsemigroup[S]
) {
    subsemigroup_is_finitely_generated(s) and s = t implies subsemigroup_is_finitely_generated(t)
} by {
    if subsemigroup_is_finitely_generated(s) and s = t {
        subsemigroup_is_finitely_generated_witness(s)
        let a: Set[S] satisfy {
            a.is_finite and subsemigroup_closure(a) = s
        }
        subsemigroup_closure(a) = t
        subsemigroup_is_finitely_generated_of_closure_eq(a, t)
        subsemigroup_is_finitely_generated(t)
    }
}

/// The closure of a finite set is a finitely generated subsemigroup.
theorem subsemigroup_closure_is_finitely_generated[S: Semigroup](a: Set[S]) {
    a.is_finite implies subsemigroup_is_finitely_generated(subsemigroup_closure(a))
} by {
    if a.is_finite {
        subsemigroup_is_finitely_generated_of_closure_eq(a, subsemigroup_closure(a))
    }
}

/// The closure of one element is a finitely generated subsemigroup.
theorem subsemigroup_closure_singleton_is_finitely_generated[S: Semigroup](a: S) {
    subsemigroup_is_finitely_generated(subsemigroup_closure(Set[S].singleton(a)))
} by {
    singleton_set_is_finite(a)
    subsemigroup_closure_is_finitely_generated(Set[S].singleton(a))
}

/// The generator belongs to the subsemigroup generated by it.
theorem subsemigroup_closure_singleton_contains[S: Semigroup](a: S) {
    subsemigroup_closure(Set[S].singleton(a)).contains(a)
} by {
    singleton_contains_eq(a, a)
    subsemigroup_closure_contains_of_set_contains(Set[S].singleton(a), a)
}

/// The empty subsemigroup is finitely generated.
theorem empty_subsemigroup_is_finitely_generated[S: Semigroup] {
    subsemigroup_is_finitely_generated(empty_subsemigroup[S])
} by {
    empty_set_is_finite[S]
    subsemigroup_closure_is_finitely_generated(Set[S].empty_set)
    subsemigroup_is_finitely_generated(subsemigroup_closure(Set[S].empty_set))
    subsemigroup_closure_empty[S]
    subsemigroup_is_finitely_generated(empty_subsemigroup[S])
}


/// Subsemigroup join is commutative.
theorem subsemigroup_sup_comm[S: Semigroup](a: Subsemigroup[S], b: Subsemigroup[S]) {
    subsemigroup_sup(a, b) = subsemigroup_sup(b, a)
} by {
    subsemigroup_subset_sup_right(b, a)
    subsemigroup_subset_sup_left(b, a)
    subsemigroup_sup_subset_of_subset_left_right(a, b, subsemigroup_sup(b, a))
    subsemigroup_subset_sup_right(a, b)
    subsemigroup_subset_sup_left(a, b)
    subsemigroup_sup_subset_of_subset_left_right(b, a, subsemigroup_sup(a, b))
    subsemigroup_subset_antisymm(subsemigroup_sup(a, b), subsemigroup_sup(b, a))
}

/// Joining a subsemigroup with itself gives the same subsemigroup.
theorem subsemigroup_sup_idempotent[S: Semigroup](a: Subsemigroup[S]) {
    subsemigroup_sup(a, a) = a
} by {
    subsemigroup_subset_refl(a)
    subsemigroup_sup_subset_of_subset_left_right(a, a, a)
    subsemigroup_subset_sup_left(a, a)
    subsemigroup_subset_antisymm(subsemigroup_sup(a, a), a)
}

/// Subsemigroup join is associative.
theorem subsemigroup_sup_assoc[S: Semigroup](
    a: Subsemigroup[S],
    b: Subsemigroup[S],
    c: Subsemigroup[S]
) {
    subsemigroup_sup(subsemigroup_sup(a, b), c) = subsemigroup_sup(a, subsemigroup_sup(b, c))
} by {
    subsemigroup_subset_sup_left(a, subsemigroup_sup(b, c))
    subsemigroup_subset_sup_right(a, subsemigroup_sup(b, c))
    subsemigroup_subset_sup_left(b, c)
    subsemigroup_subset_sup_right(b, c)
    subsemigroup_subset_trans(b, subsemigroup_sup(b, c), subsemigroup_sup(a, subsemigroup_sup(b, c)))
    subsemigroup_subset_trans(c, subsemigroup_sup(b, c), subsemigroup_sup(a, subsemigroup_sup(b, c)))
    subsemigroup_sup_subset_of_subset_left_right(a, b, subsemigroup_sup(a, subsemigroup_sup(b, c)))
    subsemigroup_sup_subset_of_subset_left_right(subsemigroup_sup(a, b), c, subsemigroup_sup(a, subsemigroup_sup(b, c)))
    subsemigroup_subset_sup_left(subsemigroup_sup(a, b), c)
    subsemigroup_subset_sup_right(subsemigroup_sup(a, b), c)
    subsemigroup_subset_sup_left(a, b)
    subsemigroup_subset_sup_right(a, b)
    subsemigroup_subset_trans(a, subsemigroup_sup(a, b), subsemigroup_sup(subsemigroup_sup(a, b), c))
    subsemigroup_subset_trans(b, subsemigroup_sup(a, b), subsemigroup_sup(subsemigroup_sup(a, b), c))
    subsemigroup_sup_subset_of_subset_left_right(b, c, subsemigroup_sup(subsemigroup_sup(a, b), c))
    subsemigroup_sup_subset_of_subset_left_right(a, subsemigroup_sup(b, c), subsemigroup_sup(subsemigroup_sup(a, b), c))
    subsemigroup_subset_antisymm(subsemigroup_sup(subsemigroup_sup(a, b), c), subsemigroup_sup(a, subsemigroup_sup(b, c)))
}
