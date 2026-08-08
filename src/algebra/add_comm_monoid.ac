from algebra.add_comm_semigroup import AddCommSemigroup
from algebra.add_monoid import AddMonoid, AddMonoidHom, is_add_monoid_hom, add_monoid_hom_zero,
    add_monoid_hom_add, identity_fn_is_add_monoid_hom, compose_is_add_monoid_hom
from data.basic.functions import identity_fn, compose, compose_assoc, compose_identity_left, compose_identity_right,
    function_eq_transport_predicate, function_eq_transport_predicate_rev

/// AddCommMonoid represents a commutative, additive monoid.
typeclass M: AddCommMonoid extends AddCommSemigroup, AddMonoid {
    // All the properties are provided by the base typeclasses.
}

/// True if a function between additive commutative monoids preserves addition and zero.
define is_add_comm_monoid_hom[A: AddCommMonoid, B: AddCommMonoid](f: A -> B) -> Bool {
    is_add_monoid_hom(f)
}

/// An additive commutative monoid homomorphism.
structure AddCommMonoidHom[A: AddCommMonoid, B: AddCommMonoid] {
    /// The mapping of the additive commutative monoid homomorphism.
    hom: A -> B
} constraint {
    is_add_comm_monoid_hom(hom)
}

/// Additive commutative monoid homomorphism extensionality.
theorem add_comm_monoid_hom_ext[A: AddCommMonoid, B: AddCommMonoid](
    f: AddCommMonoidHom[A, B],
    g: AddCommMonoidHom[A, B]
) {
    (forall(a: A) { f.hom(a) = g.hom(a) }) implies f = g
} by {
    if forall(a: A) { f.hom(a) = g.hom(a) } {
        f.hom = g.hom
    }
}

attributes AddCommMonoidHom[A: AddCommMonoid, B: AddCommMonoid] {
    /// Additive commutative monoid homomorphism extensionality from pointwise equality of the mapping.
    let ext = add_comm_monoid_hom_ext[A, B]
}

/// Equal additive commutative monoid homomorphisms have equal underlying functions.
theorem add_comm_monoid_hom_eq_hom[A: AddCommMonoid, B: AddCommMonoid](
    f: AddCommMonoidHom[A, B],
    g: AddCommMonoidHom[A, B]
) {
    f = g implies f.hom = g.hom
}

/// Equal additive commutative monoid homomorphisms have equal values at every element.
theorem add_comm_monoid_hom_eq_apply[A: AddCommMonoid, B: AddCommMonoid](
    f: AddCommMonoidHom[A, B],
    g: AddCommMonoidHom[A, B],
    a: A
) {
    f = g implies f.hom(a) = g.hom(a)
} by {
    if f = g {
        f.hom(a) = g.hom(a)
    }
}

/// Equality of additive commutative monoid homomorphisms transports predicates on homomorphisms.
theorem add_comm_monoid_hom_eq_transport_predicate[A: AddCommMonoid, B: AddCommMonoid](
    p: AddCommMonoidHom[A, B] -> Bool,
    f: AddCommMonoidHom[A, B],
    g: AddCommMonoidHom[A, B]
) {
    f = g and p(f) implies p(g)
}

/// Equality of additive commutative monoid homomorphisms transports predicates in the reverse direction.
theorem add_comm_monoid_hom_eq_transport_predicate_rev[A: AddCommMonoid, B: AddCommMonoid](
    p: AddCommMonoidHom[A, B] -> Bool,
    f: AddCommMonoidHom[A, B],
    g: AddCommMonoidHom[A, B]
) {
    f = g and p(g) implies p(f)
}

/// Equality of underlying functions determines equality of additive commutative monoid homomorphisms.
theorem add_comm_monoid_hom_eq_of_hom_eq[A: AddCommMonoid, B: AddCommMonoid](
    f: AddCommMonoidHom[A, B],
    g: AddCommMonoidHom[A, B]
) {
    f.hom = g.hom implies f = g
} by {
    if f.hom = g.hom {
        forall(a: A) {
            f.hom(a) = g.hom(a)
        }
        add_comm_monoid_hom_ext(f, g)
    }
}

/// Pointwise equality determines equality of additive commutative monoid homomorphisms.
theorem add_comm_monoid_hom_eq_of_apply_eq[A: AddCommMonoid, B: AddCommMonoid](
    f: AddCommMonoidHom[A, B],
    g: AddCommMonoidHom[A, B]
) {
    (forall(a: A) { f.hom(a) = g.hom(a) }) implies f = g
} by {
    if forall(a: A) { f.hom(a) = g.hom(a) } {
        add_comm_monoid_hom_ext(f, g)
    }
}

/// An additive commutative monoid homomorphism preserves zero.
theorem add_comm_monoid_hom_zero[A: AddCommMonoid, B: AddCommMonoid](f: AddCommMonoidHom[A, B]) {
    f.hom(A.0) = B.0
} by {
    let base: AddMonoidHom[A, B] satisfy {
        AddMonoidHom.new(f.hom) = Option.some(base)
    }
    add_monoid_hom_zero(base)
}

/// An additive commutative monoid homomorphism preserves addition.
theorem add_comm_monoid_hom_add[A: AddCommMonoid, B: AddCommMonoid](
    f: AddCommMonoidHom[A, B],
    a: A,
    b: A
) {
    f.hom(a + b) = f.hom(a) + f.hom(b)
} by {
    let base: AddMonoidHom[A, B] satisfy {
        AddMonoidHom.new(f.hom) = Option.some(base)
    }
    add_monoid_hom_add(base, a, b)
    base.hom = f.hom
}

/// Equality of functions transports the property of being an additive commutative monoid homomorphism.
theorem function_eq_transport_add_comm_monoid_hom[A: AddCommMonoid, B: AddCommMonoid](f: A -> B, g: A -> B) {
    f = g and is_add_comm_monoid_hom(f) implies is_add_comm_monoid_hom(g)
} by {
    if f = g and is_add_comm_monoid_hom(f) {
        function_eq_transport_predicate(is_add_comm_monoid_hom[A, B], f, g)
    }
}

/// Equality of functions transports the property of being an additive commutative monoid homomorphism in reverse.
theorem function_eq_transport_add_comm_monoid_hom_rev[A: AddCommMonoid, B: AddCommMonoid](f: A -> B, g: A -> B) {
    f = g and is_add_comm_monoid_hom(g) implies is_add_comm_monoid_hom(f)
} by {
    if f = g and is_add_comm_monoid_hom(g) {
        function_eq_transport_predicate_rev(is_add_comm_monoid_hom[A, B], f, g)
    }
}

/// The identity function on an additive commutative monoid preserves additive commutative monoid structure.
theorem identity_fn_is_add_comm_monoid_hom[A: AddCommMonoid] {
    is_add_comm_monoid_hom(identity_fn[A])
} by {
    identity_fn_is_add_monoid_hom[A]
}

/// The composition of additive commutative monoid homomorphisms preserves structure.
theorem compose_is_add_comm_monoid_hom[A: AddCommMonoid, B: AddCommMonoid, C: AddCommMonoid](
    f: B -> C,
    g: A -> B
) {
    is_add_comm_monoid_hom(f) and is_add_comm_monoid_hom(g) implies
    is_add_comm_monoid_hom(compose(f, g))
} by {
    if is_add_comm_monoid_hom(f) and is_add_comm_monoid_hom(g) {
        is_add_monoid_hom(g)
        compose_is_add_monoid_hom(f, g)
        is_add_monoid_hom(compose(f, g))
    }
}

/// The identity homomorphism on an additive commutative monoid.
let identity_add_comm_monoid_hom[A: AddCommMonoid]: AddCommMonoidHom[A, A] satisfy {
    AddCommMonoidHom.new(identity_fn[A]) = Option.some(identity_add_comm_monoid_hom)
}

/// The underlying function of the identity additive commutative monoid homomorphism.
theorem identity_add_comm_monoid_hom_hom[A: AddCommMonoid] {
    identity_add_comm_monoid_hom[A].hom = identity_fn[A]
}

/// The composition of two additive commutative monoid homomorphisms.
let compose_add_comm_monoid_hom[A: AddCommMonoid, B: AddCommMonoid, C: AddCommMonoid](f: AddCommMonoidHom[B, C], g: AddCommMonoidHom[A, B]) -> result: AddCommMonoidHom[A, C] satisfy {
    AddCommMonoidHom.new(compose(f.hom, g.hom)) = Option.some(result)
} by {
    compose_is_add_comm_monoid_hom(f.hom, g.hom)
}

/// The underlying function of a composition of additive commutative monoid homomorphisms.
theorem compose_add_comm_monoid_hom_hom[A: AddCommMonoid, B: AddCommMonoid, C: AddCommMonoid](
    f: AddCommMonoidHom[B, C],
    g: AddCommMonoidHom[A, B]
) {
    compose_add_comm_monoid_hom(f, g).hom = compose(f.hom, g.hom)
}

/// Composition of additive commutative monoid homomorphisms is associative.
theorem compose_add_comm_monoid_hom_assoc[A: AddCommMonoid, B: AddCommMonoid, C: AddCommMonoid, D: AddCommMonoid](
    f: AddCommMonoidHom[C, D],
    g: AddCommMonoidHom[B, C],
    h: AddCommMonoidHom[A, B]
) {
    compose_add_comm_monoid_hom(compose_add_comm_monoid_hom(f, g), h) =
    compose_add_comm_monoid_hom(f, compose_add_comm_monoid_hom(g, h))
} by {
    let lhs = compose_add_comm_monoid_hom(compose_add_comm_monoid_hom(f, g), h)
    let rhs = compose_add_comm_monoid_hom(f, compose_add_comm_monoid_hom(g, h))
    lhs.hom = compose(compose_add_comm_monoid_hom(f, g).hom, h.hom)
    compose_add_comm_monoid_hom(f, g).hom = compose(f.hom, g.hom)
    lhs.hom = compose(compose(f.hom, g.hom), h.hom)
    rhs.hom = compose(f.hom, compose_add_comm_monoid_hom(g, h).hom)
    compose_add_comm_monoid_hom(g, h).hom = compose(g.hom, h.hom)
    rhs.hom = compose(f.hom, compose(g.hom, h.hom))
    compose_assoc(f.hom, g.hom, h.hom)
    lhs.hom = rhs.hom
}

/// Composing the identity additive commutative monoid homomorphism on the left leaves a homomorphism unchanged.
theorem compose_add_comm_monoid_hom_identity_left[A: AddCommMonoid, B: AddCommMonoid](
    f: AddCommMonoidHom[A, B]
) {
    compose_add_comm_monoid_hom(identity_add_comm_monoid_hom[B], f) = f
} by {
    let lhs = compose_add_comm_monoid_hom(identity_add_comm_monoid_hom[B], f)
    compose_identity_left(f.hom)
}

/// Composing the identity additive commutative monoid homomorphism on the right leaves a homomorphism unchanged.
theorem compose_add_comm_monoid_hom_identity_right[A: AddCommMonoid, B: AddCommMonoid](
    f: AddCommMonoidHom[A, B]
) {
    compose_add_comm_monoid_hom(f, identity_add_comm_monoid_hom[A]) = f
} by {
    let lhs = compose_add_comm_monoid_hom(f, identity_add_comm_monoid_hom[A])
    compose_identity_right(f.hom)
    lhs.hom = f.hom
}
