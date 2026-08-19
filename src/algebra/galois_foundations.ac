/// Galois theory foundations: the quadratic extension Q(√2).
///
/// This file records the concrete first steps of Galois theory for the
/// simplest nontrivial example, the quadratic extension Q(√2) of the
/// rationals:
///
///   1. The square root of two and the roots of `X^2 - 2`: both `√2` and
///      `-√2` square to two, so both are roots of `X^2 - 2` over the reals.
///   2. The minimal polynomial of `√2` over Q: `X^2 - 2` has no rational
///      root (a rational root in lowest terms would have a numerator
///      dividing two and a denominator dividing one, contradicting that no
///      integer squares to two; equivalently, `p^2 = 2 q^2` has no positive
///      integer solution, the classical irrationality of `√2`).
///   3. The automorphism group of Q(√2): the map `a + b√2 ↦ a - b√2`
///      preserves addition and multiplication (verified for concrete
///      elements), and complex conjugation, the library's conjugation
///      machinery, fixes Q(√2) pointwise because Q(√2) sits inside the
///      reals.
///
/// The fundamental theorem of Galois theory itself is stated only as a
/// comment at the end of the file: the full lattice correspondence between
/// intermediate fields and subgroups of the automorphism group needs
/// subfield and automorphism-group machinery that is not yet in the library.

from nat import Nat
from int import Int
from nat import gcd_mult_right, divides_gcd, gcd_of_prime, mul_to_zero, mul_cancel_left,
    divisor_lt
from rat import Rat, one_is_pos, div_from_int, mul_cancels_div, from_int_cancel, from_int_zero,
    denom_nonzero, mul_int_eq_int_mul
from int import abs, abs_mul, abs_from_nat, abs_zero_imp_zero
from real import Real, add_assoc, two, two_nonzero, sqrt_mul_self, sqrt_value_nonneg, from_rat_pos,
    pos_gt_zero, gt_zero_imp_pos, lt_add_pos, lt_trans, neg_distrib, mul_neg_left, mul_neg_right,
    real_mul_comm, mul_assoc, mul_distrib_right, mul_distrib_left, sub_zero_imp_eq
from order import lt_imp_lte
from algebra.well_founded import nat_lt_relation, nat_lt_relation_induction_at
from semiring import Semiring
from algebra.add_group import inverse_left, right_cancel
from algebra.add_comm_group import sub_add_sub
from algebra.field.field import field_mul_eq_zero
from algebra.field.field_hom import FieldHom, field_hom_add, field_hom_mul
from algebra.ring.ring import mul_neg_neg, mul_sub_right
from polynomial import Polynomial, polynomial_constant, polynomial_monomial, polynomial_eval,
    polynomial_eval_add, polynomial_eval_constant, polynomial_eval_bound, polynomial_eval_bound_eq_coeff_eval,
    polynomial_eval_eq_eval_bound_of_support_bounded, polynomial_support_bounded_by,
    polynomial_monomial_coeff_of_ne, polynomial_monomial_coeff_self,
    coeff_eval, coeff_tail
from complex import Complex, real_add_lifts, real_mul_lifts, complex_conj_field_hom,
    complex_conj_field_hom_from_real, polynomial_monomial_support_bounded_by_suc
from number_theory import nat_two_prime

numerals Nat
numerals Int
numerals Rat
numerals Real

// ---------------------------------------------------------------------------
// 0. Horner evaluation lemmas
// ---------------------------------------------------------------------------

/// Horner evaluation at a successor bound unfolds one coefficient.
theorem coeff_eval_suc_unfold[R: Semiring](c: Nat -> R, x: R, n: Nat) {
    coeff_eval(c, x, n.suc) = c(Nat.0) + x * coeff_eval(coeff_tail(c), x, n)
} by {
}

/// Horner evaluation at bound zero is zero.
theorem coeff_eval_nat_zero[R: Semiring](c: Nat -> R, x: R) {
    coeff_eval(c, x, Nat.0) = R.0
} by {
}

// ---------------------------------------------------------------------------
// 1. The square root of two
// ---------------------------------------------------------------------------

/// The real number one is positive.
theorem real_one_pos {
    Real.1 > Real.0
} by {
    one_is_pos
    Rat.1.is_positive
    from_rat_pos(Rat.1)
    Real.from_rat(Rat.1).is_positive
    Real.1 = Real.from_rat(Rat.1)
    Real.1.is_positive
    pos_gt_zero(Real.1)
    Real.1 > Real.0
}

/// The real number two is positive.
theorem two_positive_gt {
    two > Real.0
} by {
    two = Real.1 + Real.1
    real_one_pos
    Real.1 > Real.0
    gt_zero_imp_pos(Real.1)
    Real.1.is_positive
    lt_add_pos(Real.1, Real.1)
    Real.1 < Real.1 + Real.1
    Real.1 < two
    lt_trans(Real.0, Real.1, two)
    Real.0 < two
    two > Real.0
}

/// The real number two is nonnegative.
theorem two_nonneg {
    two >= Real.0
} by {
    two_positive_gt
    two > Real.0
    lt_imp_lte(Real.0, two)
    Real.0 <= two
}

/// There exists a real square root of two.
theorem sqrt_two_exists {
    exists(r: Real) {
        two.sqrt = Option.some(r) and r * r = two
    }
} by {
    two_nonneg
    sqrt_mul_self(two)
    exists(y: Real) {
        two.sqrt = Option.some(y) and y * y = two
    }
    let (y: Real) satisfy {
        two.sqrt = Option.some(y) and y * y = two
    }
    exists(witness: Real) {
        witness = y and two.sqrt = Option.some(witness) and witness * witness = two
    }
}

/// The positive square root of two, `√2`, a real number whose square is two.
let sqrt_two: Real satisfy {
    two.sqrt = Option.some(sqrt_two) and sqrt_two * sqrt_two = two
}

/// The square of the square root of two is two: `(√2)² = 2`.
theorem sqrt_two_square {
    sqrt_two * sqrt_two = two
} by {
}

/// The square root of two is nonnegative.
theorem sqrt_two_nonneg {
    sqrt_two >= Real.0
} by {
    two_nonneg
    sqrt_value_nonneg(two, sqrt_two)
    two.sqrt = Option.some(sqrt_two) and sqrt_two * sqrt_two = two
    two.sqrt = Option.some(sqrt_two)
    sqrt_two >= Real.0
}

/// The square of the negative of the square root of two is two: `(-√2)² = 2`.
///
/// This is the sign symmetry: the two roots of `X^2 - 2` are negatives of
/// each other.
theorem neg_sqrt_two_square {
    -sqrt_two * -sqrt_two = two
} by {
    mul_neg_neg[Real](sqrt_two, sqrt_two)
    -sqrt_two * -sqrt_two = sqrt_two * sqrt_two
    sqrt_two * sqrt_two = two
    -sqrt_two * -sqrt_two = two
}

// ---------------------------------------------------------------------------
// 2. The roots of X^2 - 2 over the reals
// ---------------------------------------------------------------------------

/// The polynomial `X^2 - 2` with real coefficients.
let real_x_squared_minus_two: Polynomial[Real] =
    polynomial_constant(-two) + polynomial_monomial(Nat.2, Real.1)

/// Evaluation of the degree-two monomial over the reals.
theorem real_polynomial_eval_monomial_two(r: Real, x: Real) {
    polynomial_eval(Polynomial[Real].monomial(Nat.2, r), x) = r * x * x
} by {
    polynomial_eval_bound_eq_coeff_eval(Polynomial[Real].monomial(Nat.2, r), x,
        Nat.0.suc.suc.suc)
    coeff_eval_suc_unfold[Real](Polynomial[Real].monomial(Nat.2, r).coeff, x, Nat.0.suc.suc)
    coeff_eval(Polynomial[Real].monomial(Nat.2, r).coeff, x, Nat.0.suc.suc.suc) =
        Polynomial[Real].monomial(Nat.2, r).coeff(Nat.0) +
        x * coeff_eval(coeff_tail(Polynomial[Real].monomial(Nat.2, r).coeff), x, Nat.0.suc.suc)
    polynomial_monomial_coeff_of_ne[Real](Nat.2, r, Nat.0)
    Nat.0 != Nat.2
    Polynomial[Real].monomial(Nat.2, r).coeff(Nat.0) = Real.0
    coeff_eval_suc_unfold[Real](coeff_tail(Polynomial[Real].monomial(Nat.2, r).coeff), x,
        Nat.0.suc)
    coeff_eval(coeff_tail(Polynomial[Real].monomial(Nat.2, r).coeff), x, Nat.0.suc.suc) =
        coeff_tail(Polynomial[Real].monomial(Nat.2, r).coeff)(Nat.0) +
        x * coeff_eval(coeff_tail(coeff_tail(Polynomial[Real].monomial(Nat.2, r).coeff)), x,
            Nat.0.suc)
    coeff_tail(Polynomial[Real].monomial(Nat.2, r).coeff)(Nat.0) =
        Polynomial[Real].monomial(Nat.2, r).coeff(Nat.1)
    polynomial_monomial_coeff_of_ne[Real](Nat.2, r, Nat.1)
    Nat.1 != Nat.2
    Polynomial[Real].monomial(Nat.2, r).coeff(Nat.1) = Real.0
    coeff_eval_suc_unfold[Real](
        coeff_tail(coeff_tail(Polynomial[Real].monomial(Nat.2, r).coeff)), x, Nat.0)
    coeff_eval(coeff_tail(coeff_tail(Polynomial[Real].monomial(Nat.2, r).coeff)), x, Nat.0.suc) =
        coeff_tail(coeff_tail(Polynomial[Real].monomial(Nat.2, r).coeff))(Nat.0) +
        x * coeff_eval(coeff_tail(coeff_tail(coeff_tail(
            Polynomial[Real].monomial(Nat.2, r).coeff))), x, Nat.0)
    coeff_tail(coeff_tail(Polynomial[Real].monomial(Nat.2, r).coeff))(Nat.0) =
        Polynomial[Real].monomial(Nat.2, r).coeff(Nat.2)
    polynomial_monomial_coeff_self(Nat.2, r)
    Polynomial[Real].monomial(Nat.2, r).coeff(Nat.2) = r
    coeff_eval_nat_zero[Real](
        coeff_tail(coeff_tail(coeff_tail(Polynomial[Real].monomial(Nat.2, r).coeff))), x)
    coeff_eval(coeff_tail(coeff_tail(coeff_tail(
        Polynomial[Real].monomial(Nat.2, r).coeff))), x, Nat.0) = Real.0
    coeff_eval(Polynomial[Real].monomial(Nat.2, r).coeff, x, Nat.0.suc.suc.suc) =
        Real.0 + x * (Real.0 + x * (r + x * Real.0))
    Real.0 + x * (Real.0 + x * (r + x * Real.0)) = r * x * x
    coeff_eval(Polynomial[Real].monomial(Nat.2, r).coeff, x, Nat.0.suc.suc.suc) = r * x * x
    polynomial_eval_bound(Polynomial[Real].monomial(Nat.2, r), x, Nat.0.suc.suc.suc) =
        coeff_eval(Polynomial[Real].monomial(Nat.2, r).coeff, x, Nat.0.suc.suc.suc)
    polynomial_eval_bound(Polynomial[Real].monomial(Nat.2, r), x, Nat.0.suc.suc.suc) = r * x * x
    polynomial_monomial_support_bounded_by_suc[Real](Nat.2, r)
    polynomial_support_bounded_by(Polynomial[Real].monomial(Nat.2, r), Nat.0.suc.suc.suc)
    polynomial_eval_eq_eval_bound_of_support_bounded(Polynomial[Real].monomial(Nat.2, r), x,
        Nat.0.suc.suc.suc)
    polynomial_eval(Polynomial[Real].monomial(Nat.2, r), x) =
        polynomial_eval_bound(Polynomial[Real].monomial(Nat.2, r), x, Nat.0.suc.suc.suc)
    polynomial_eval(Polynomial[Real].monomial(Nat.2, r), x) = r * x * x
}

/// The value of `X^2 - 2` at a real number `x` is `x² - 2`.
theorem real_x_squared_minus_two_eval(x: Real) {
    polynomial_eval(real_x_squared_minus_two, x) = x * x - two
} by {
    polynomial_eval_add(polynomial_constant(-two), polynomial_monomial(Nat.2, Real.1), x)
    polynomial_eval(polynomial_constant(-two) + polynomial_monomial(Nat.2, Real.1), x) =
        polynomial_eval(polynomial_constant(-two), x) +
        polynomial_eval(polynomial_monomial(Nat.2, Real.1), x)
    polynomial_eval_constant(-two, x)
    polynomial_eval(polynomial_constant(-two), x) = -two
    real_polynomial_eval_monomial_two(Real.1, x)
    polynomial_eval(polynomial_monomial(Nat.2, Real.1), x) = Real.1 * x * x
    Real.1 * x * x = x * x
    polynomial_eval(real_x_squared_minus_two, x) = -two + x * x
    -two + x * x = x * x - two
    polynomial_eval(real_x_squared_minus_two, x) = x * x - two
}

/// The square root of two is a root of `X^2 - 2`.
theorem sqrt_two_is_root_of_x_squared_minus_two {
    polynomial_eval(real_x_squared_minus_two, sqrt_two) = Real.0
} by {
    real_x_squared_minus_two_eval(sqrt_two)
    polynomial_eval(real_x_squared_minus_two, sqrt_two) = sqrt_two * sqrt_two - two
    sqrt_two_square
    sqrt_two * sqrt_two = two
    two - two = Real.0
    sqrt_two * sqrt_two - two = Real.0
    polynomial_eval(real_x_squared_minus_two, sqrt_two) = Real.0
}

/// The negative of the square root of two is a root of `X^2 - 2`.
theorem neg_sqrt_two_is_root_of_x_squared_minus_two {
    polynomial_eval(real_x_squared_minus_two, -sqrt_two) = Real.0
} by {
    real_x_squared_minus_two_eval(-sqrt_two)
    polynomial_eval(real_x_squared_minus_two, -sqrt_two) = (-sqrt_two) * (-sqrt_two) - two
    neg_sqrt_two_square
    -sqrt_two * -sqrt_two = two
    two - two = Real.0
    (-sqrt_two) * (-sqrt_two) - two = Real.0
    polynomial_eval(real_x_squared_minus_two, -sqrt_two) = Real.0
}

/// The two roots `√2` and `-√2` of `X^2 - 2` are distinct.
theorem sqrt_two_ne_neg_sqrt_two {
    sqrt_two != -sqrt_two
} by {
    if sqrt_two = -sqrt_two {
        sqrt_two + sqrt_two = two * sqrt_two
        sqrt_two + sqrt_two = -sqrt_two + sqrt_two
        inverse_left[Real](sqrt_two)
        -sqrt_two + sqrt_two = Real.0
        sqrt_two + sqrt_two = Real.0
        two * sqrt_two = Real.0
        field_mul_eq_zero[Real](two, sqrt_two)
        two = Real.0 or sqrt_two = Real.0
        two_nonzero
        two != Real.0
        sqrt_two = Real.0
        sqrt_two * sqrt_two = Real.0 * Real.0
        Real.0 * Real.0 = Real.0
        sqrt_two * sqrt_two = Real.0
        sqrt_two_square
        sqrt_two * sqrt_two = two
        two = Real.0
        two_nonzero
        false
    }
}

/// The difference of squares: `(x - y)(x + y) = x² - y²`.
theorem diff_sq(x: Real, y: Real) {
    (x - y) * (x + y) = x * x - y * y
} by {
    mul_sub_right[Real](x, y, x + y)
    (x - y) * (x + y) = x * (x + y) - y * (x + y)
    mul_distrib_right(x, x, y)
    x * (x + y) = x * x + x * y
    mul_distrib_right(y, x, y)
    y * (x + y) = y * x + y * y
    (x - y) * (x + y) = (x * x + x * y) - (y * x + y * y)
    real_mul_comm(x, y)
    x * y = y * x
    (x * x + x * y) - (y * x + y * y) = (x * x + x * y) - (x * y + y * y)
    sub_add_sub(x * x, x * y, x * y, y * y)
    (x * x + x * y) - (x * y + y * y) = (x * x - x * y) + (x * y - y * y)
    add_assoc(x * x, -(x * y), x * y - y * y)
    (x * x - x * y) + (x * y - y * y) = x * x + (-(x * y) + (x * y - y * y))
    add_assoc(-(x * y), x * y, -(y * y))
    (-(x * y) + x * y) + -(y * y) = -(x * y) + (x * y + -(y * y))
    inverse_left(x * y)
    -(x * y) + x * y = Real.0
    x * x + (-(x * y) + (x * y - y * y)) = x * x + -(y * y)
    x * x + -(y * y) = x * x - y * y
    (x * x + x * y) - (x * y + y * y) = x * x - y * y
    (x - y) * (x + y) = x * x - y * y
}

/// A real number whose square is two is either `√2` or `-√2`.
///
/// From `x² = 2` the factorisation `(x - √2)(x + √2) = 0` holds, and a
/// field has no zero divisors.
theorem real_square_two_imp_eq_or_neg(x: Real) {
    x * x = two implies x = sqrt_two or x = -sqrt_two
} by {
    if x * x = two {
        diff_sq(x, sqrt_two)
        (x - sqrt_two) * (x + sqrt_two) = x * x - sqrt_two * sqrt_two
        sqrt_two_square
        sqrt_two * sqrt_two = two
        (x - sqrt_two) * (x + sqrt_two) = x * x - two
        x * x - two = Real.0
        (x - sqrt_two) * (x + sqrt_two) = Real.0
        field_mul_eq_zero[Real](x - sqrt_two, x + sqrt_two)
        x - sqrt_two = Real.0 or x + sqrt_two = Real.0
        if x - sqrt_two = Real.0 {
            sub_zero_imp_eq(x, sqrt_two)
            x = sqrt_two
            x = sqrt_two or x = -sqrt_two
        }
        if x + sqrt_two = Real.0 {
            inverse_left[Real](sqrt_two)
            -sqrt_two + sqrt_two = Real.0
            x + sqrt_two = -sqrt_two + sqrt_two
            right_cancel(sqrt_two, x, -sqrt_two)
            x = -sqrt_two
            x = sqrt_two or x = -sqrt_two
        }
        x = sqrt_two or x = -sqrt_two
    }
}

/// A root of `X^2 - 2` over the reals is either `√2` or `-√2`.
theorem real_x_squared_minus_two_root_forward(x: Real) {
    polynomial_eval(real_x_squared_minus_two, x) = Real.0 implies
        (x = sqrt_two or x = -sqrt_two)
} by {
    if polynomial_eval(real_x_squared_minus_two, x) = Real.0 {
        real_x_squared_minus_two_eval(x)
        polynomial_eval(real_x_squared_minus_two, x) = x * x - two
        x * x - two = Real.0
        sub_zero_imp_eq(x * x, two)
        x * x = two
        real_square_two_imp_eq_or_neg(x)
        x = sqrt_two or x = -sqrt_two
    }
}

/// Both `√2` and `-√2` are roots of `X^2 - 2`.
///
/// Together with `real_x_squared_minus_two_root_forward` this shows that the
/// roots of `X^2 - 2` over the reals are exactly `√2` and `-√2`.
theorem real_x_squared_minus_two_root_backward(x: Real) {
    (x = sqrt_two or x = -sqrt_two) implies
        polynomial_eval(real_x_squared_minus_two, x) = Real.0
} by {
    if x = sqrt_two or x = -sqrt_two {
        if x = sqrt_two {
            sqrt_two_is_root_of_x_squared_minus_two
            polynomial_eval(real_x_squared_minus_two, sqrt_two) = Real.0
            polynomial_eval(real_x_squared_minus_two, x) = Real.0
        } else {
            neg_sqrt_two_is_root_of_x_squared_minus_two
            polynomial_eval(real_x_squared_minus_two, -sqrt_two) = Real.0
            polynomial_eval(real_x_squared_minus_two, x) = Real.0
        }
        polynomial_eval(real_x_squared_minus_two, x) = Real.0
    }
}

// ---------------------------------------------------------------------------
// 3. The minimal polynomial of √2 over Q: X^2 - 2
// ---------------------------------------------------------------------------

/// The polynomial `X^2 - 2` with rational coefficients.
let rat_x_squared_minus_two: Polynomial[Rat] =
    polynomial_constant(-Rat.2) + polynomial_monomial(Nat.2, Rat.1)

/// Evaluation of the degree-two monomial over the rationals.
theorem rat_polynomial_eval_monomial_two(r: Rat, x: Rat) {
    polynomial_eval(Polynomial[Rat].monomial(Nat.2, r), x) = r * x * x
} by {
    polynomial_eval_bound_eq_coeff_eval(Polynomial[Rat].monomial(Nat.2, r), x,
        Nat.0.suc.suc.suc)
    coeff_eval_suc_unfold[Rat](Polynomial[Rat].monomial(Nat.2, r).coeff, x, Nat.0.suc.suc)
    coeff_eval(Polynomial[Rat].monomial(Nat.2, r).coeff, x, Nat.0.suc.suc.suc) =
        Polynomial[Rat].monomial(Nat.2, r).coeff(Nat.0) +
        x * coeff_eval(coeff_tail(Polynomial[Rat].monomial(Nat.2, r).coeff), x, Nat.0.suc.suc)
    polynomial_monomial_coeff_of_ne[Rat](Nat.2, r, Nat.0)
    Nat.0 != Nat.2
    Polynomial[Rat].monomial(Nat.2, r).coeff(Nat.0) = Rat.0
    coeff_eval_suc_unfold[Rat](coeff_tail(Polynomial[Rat].monomial(Nat.2, r).coeff), x,
        Nat.0.suc)
    coeff_eval(coeff_tail(Polynomial[Rat].monomial(Nat.2, r).coeff), x, Nat.0.suc.suc) =
        coeff_tail(Polynomial[Rat].monomial(Nat.2, r).coeff)(Nat.0) +
        x * coeff_eval(coeff_tail(coeff_tail(Polynomial[Rat].monomial(Nat.2, r).coeff)), x,
            Nat.0.suc)
    coeff_tail(Polynomial[Rat].monomial(Nat.2, r).coeff)(Nat.0) =
        Polynomial[Rat].monomial(Nat.2, r).coeff(Nat.1)
    polynomial_monomial_coeff_of_ne[Rat](Nat.2, r, Nat.1)
    Nat.1 != Nat.2
    Polynomial[Rat].monomial(Nat.2, r).coeff(Nat.1) = Rat.0
    coeff_eval_suc_unfold[Rat](
        coeff_tail(coeff_tail(Polynomial[Rat].monomial(Nat.2, r).coeff)), x, Nat.0)
    coeff_eval(coeff_tail(coeff_tail(Polynomial[Rat].monomial(Nat.2, r).coeff)), x, Nat.0.suc) =
        coeff_tail(coeff_tail(Polynomial[Rat].monomial(Nat.2, r).coeff))(Nat.0) +
        x * coeff_eval(coeff_tail(coeff_tail(coeff_tail(
            Polynomial[Rat].monomial(Nat.2, r).coeff))), x, Nat.0)
    coeff_tail(coeff_tail(Polynomial[Rat].monomial(Nat.2, r).coeff))(Nat.0) =
        Polynomial[Rat].monomial(Nat.2, r).coeff(Nat.2)
    polynomial_monomial_coeff_self(Nat.2, r)
    Polynomial[Rat].monomial(Nat.2, r).coeff(Nat.2) = r
    coeff_eval_nat_zero[Rat](
        coeff_tail(coeff_tail(coeff_tail(Polynomial[Rat].monomial(Nat.2, r).coeff))), x)
    coeff_eval(coeff_tail(coeff_tail(coeff_tail(
        Polynomial[Rat].monomial(Nat.2, r).coeff))), x, Nat.0) = Rat.0
    coeff_eval(Polynomial[Rat].monomial(Nat.2, r).coeff, x, Nat.0.suc.suc.suc) =
        Rat.0 + x * (Rat.0 + x * (r + x * Rat.0))
    Rat.0 + x * (Rat.0 + x * (r + x * Rat.0)) = r * x * x
    coeff_eval(Polynomial[Rat].monomial(Nat.2, r).coeff, x, Nat.0.suc.suc.suc) = r * x * x
    polynomial_eval_bound(Polynomial[Rat].monomial(Nat.2, r), x, Nat.0.suc.suc.suc) =
        coeff_eval(Polynomial[Rat].monomial(Nat.2, r).coeff, x, Nat.0.suc.suc.suc)
    polynomial_eval_bound(Polynomial[Rat].monomial(Nat.2, r), x, Nat.0.suc.suc.suc) = r * x * x
    polynomial_monomial_support_bounded_by_suc[Rat](Nat.2, r)
    polynomial_support_bounded_by(Polynomial[Rat].monomial(Nat.2, r), Nat.0.suc.suc.suc)
    polynomial_eval_eq_eval_bound_of_support_bounded(Polynomial[Rat].monomial(Nat.2, r), x,
        Nat.0.suc.suc.suc)
    polynomial_eval(Polynomial[Rat].monomial(Nat.2, r), x) =
        polynomial_eval_bound(Polynomial[Rat].monomial(Nat.2, r), x, Nat.0.suc.suc.suc)
    polynomial_eval(Polynomial[Rat].monomial(Nat.2, r), x) = r * x * x
}

/// The value of `X^2 - 2` at a rational point `x` is `x² - 2`.
theorem rat_x_squared_minus_two_eval(x: Rat) {
    polynomial_eval(rat_x_squared_minus_two, x) = x * x - Rat.2
} by {
    polynomial_eval_add(polynomial_constant(-Rat.2), polynomial_monomial(Nat.2, Rat.1), x)
    polynomial_eval(polynomial_constant(-Rat.2) + polynomial_monomial(Nat.2, Rat.1), x) =
        polynomial_eval(polynomial_constant(-Rat.2), x) +
        polynomial_eval(polynomial_monomial(Nat.2, Rat.1), x)
    polynomial_eval_constant(-Rat.2, x)
    polynomial_eval(polynomial_constant(-Rat.2), x) = -Rat.2
    rat_polynomial_eval_monomial_two(Rat.1, x)
    polynomial_eval(polynomial_monomial(Nat.2, Rat.1), x) = Rat.1 * x * x
    Rat.1 * x * x = x * x
    polynomial_eval(rat_x_squared_minus_two, x) = -Rat.2 + x * x
    -Rat.2 + x * x = x * x - Rat.2
    polynomial_eval(rat_x_squared_minus_two, x) = x * x - Rat.2
}

// ---------------------------------------------------------------------------
// 3a. The Diophantine equation x² = 2y² (the irrationality of √2)
// ---------------------------------------------------------------------------

/// The square of a doubled natural number: `(2x)² = 4x²`.
theorem nat_sq_double(x: Nat) {
    (Nat.2 * x) * (Nat.2 * x) = Nat.4 * (x * x)
} by {
    (Nat.2 * x) * (Nat.2 * x) = Nat.2 * x * Nat.2 * x
    Nat.2 * x * Nat.2 * x = Nat.2 * Nat.2 * x * x
    Nat.2 * Nat.2 = Nat.4
    Nat.2 * Nat.2 * x * x = Nat.4 * (x * x)
    (Nat.2 * x) * (Nat.2 * x) = Nat.4 * (x * x)
}

/// Euclid's lemma: if `a` is coprime to `b` and divides `b * c`, then `a` divides `c`.
theorem nat_euclid_coprime_divides(a: Nat, b: Nat, c: Nat) {
    a.coprime(b) and a.divides(b * c) implies a.divides(c)
} by {
    if a.coprime(b) and a.divides(b * c) {
        a.gcd(b) = Nat.1
        gcd_mult_right(a, b, c)
        a.gcd(b) * c = (a * c).gcd(b * c)
        Nat.1 * c = (a * c).gcd(b * c)
        c = (a * c).gcd(b * c)
        a.divides(a * c)
        a.divides(b * c)
        divides_gcd(a, a * c, b * c)
        a.divides((a * c).gcd(b * c))
        a.divides(c)
    }
}

/// If two divides a square, it divides the root (two is prime).
theorem nat_two_divides_square_imp_two_divides(x: Nat) {
    Nat.2.divides(x * x) implies Nat.2.divides(x)
} by {
    if Nat.2.divides(x * x) {
        nat_two_prime
        gcd_of_prime(Nat.2, x)
        if Nat.2.gcd(x) = Nat.1 {
            Nat.2.coprime(x)
            nat_euclid_coprime_divides(Nat.2, x, x)
            Nat.2.divides(x)
        } else {
            Nat.2.divides(x)
        }
    }
}

/// Infinite descent: a nonzero solution of `x² = 2y²` yields a smaller one.
///
/// From `x² = 2y²` both coordinates are even, `x = 2a` and `y = 2b`, and
/// then `a² = 2b²` with `a < x`.
theorem nat_sq_eq_two_sq_descent(x: Nat, y: Nat) {
    x != Nat.0 and x * x = Nat.2 * (y * y) implies
        exists(a: Nat, b: Nat) {
            a < x and Nat.2 * a = x and a * a = Nat.2 * (b * b)
        }
} by {
    if x != Nat.0 and x * x = Nat.2 * (y * y) {
        Nat.2.divides(Nat.2 * (y * y))
        Nat.2 * (y * y) = x * x
        Nat.2.divides(x * x)
        nat_two_divides_square_imp_two_divides(x)
        Nat.2.divides(x)
        let (a: Nat) satisfy { Nat.2 * a = x }
        mul_to_zero(Nat.2, a)
        if Nat.2 * a != Nat.0 {
            a != Nat.0
        }
        a != Nat.0
        a * Nat.2 = Nat.2 * a
        a * Nat.2 = x
        Nat.1 < Nat.2
        divisor_lt(a, Nat.2, x)
        a < x
        Nat.2 * a = x
        x * x = (Nat.2 * a) * (Nat.2 * a)
        x * x = Nat.2 * (y * y)
        (Nat.2 * a) * (Nat.2 * a) = Nat.2 * (y * y)
        nat_sq_double(a)
        (Nat.2 * a) * (Nat.2 * a) = Nat.4 * (a * a)
        Nat.4 * (a * a) = Nat.2 * (y * y)
        Nat.2 * (Nat.2 * (a * a)) = Nat.2 * (y * y)
        mul_cancel_left(Nat.2, Nat.2 * (a * a), y * y)
        Nat.2 * (a * a) = y * y
        Nat.2.divides(Nat.2 * (a * a))
        Nat.2 * (a * a) = y * y
        Nat.2.divides(y * y)
        nat_two_divides_square_imp_two_divides(y)
        Nat.2.divides(y)
        let (b: Nat) satisfy { Nat.2 * b = y }
        Nat.2 * b = y
        y * y = (Nat.2 * b) * (Nat.2 * b)
        Nat.2 * (a * a) = y * y
        Nat.2 * (a * a) = (Nat.2 * b) * (Nat.2 * b)
        nat_sq_double(b)
        (Nat.2 * b) * (Nat.2 * b) = Nat.4 * (b * b)
        Nat.2 * (a * a) = Nat.4 * (b * b)
        Nat.2 * (a * a) = Nat.2 * (Nat.2 * (b * b))
        mul_cancel_left(Nat.2, a * a, Nat.2 * (b * b))
        a * a = Nat.2 * (b * b)
        a < x and Nat.2 * a = x and a * a = Nat.2 * (b * b)
        exists(a2: Nat, b2: Nat) {
            a2 < x and Nat.2 * a2 = x and a2 * a2 = Nat.2 * (b2 * b2)
        }
    }
}

/// The infinite descent of `nat_sq_eq_two_sq_descent`, run by well-founded
/// induction on the first coordinate: `x² = 2y²` forces `x = 0`.
theorem nat_sq_eq_two_sq_zero(x: Nat) {
    forall(y: Nat) { x * x = Nat.2 * (y * y) implies x = Nat.0 }
} by {
    let f: Nat -> Bool = function(t: Nat) {
        forall(y: Nat) { t * t = Nat.2 * (y * y) implies t = Nat.0 }
    }
    forall(z: Nat) {
        if forall(k: Nat) { nat_lt_relation(k, z) implies f(k) } {
            forall(y: Nat) {
                if z * z = Nat.2 * (y * y) {
                    if z = Nat.0 {
                        z = Nat.0
                    } else {
                        z != Nat.0 and z * z = Nat.2 * (y * y)
                        nat_sq_eq_two_sq_descent(z, y)
                        let (a: Nat, b: Nat) satisfy {
                            a < z and Nat.2 * a = z and a * a = Nat.2 * (b * b)
                        }
                        nat_lt_relation(a, z) = (a < z)
                        nat_lt_relation(a, z)
                        forall(k: Nat) { nat_lt_relation(k, z) implies f(k) }
                        nat_lt_relation(a, z) implies f(a)
                        f(a)
                        f(a) =
                            forall(w: Nat) { a * a = Nat.2 * (w * w) implies a = Nat.0 }
                        forall(w: Nat) { a * a = Nat.2 * (w * w) implies a = Nat.0 }
                        a * a = Nat.2 * (b * b) implies a = Nat.0
                        a = Nat.0
                        Nat.2 * a = z
                        Nat.2 * Nat.0 = Nat.0
                        z = Nat.0
                        false
                    }
                }
            }
            forall(y: Nat) { z * z = Nat.2 * (y * y) implies z = Nat.0 }
            f(z) = forall(w: Nat) { z * z = Nat.2 * (w * w) implies z = Nat.0 }
            f(z)
        }
    }
    nat_lt_relation_induction_at(f, x)
    f(x)
    f(x) = forall(y: Nat) { x * x = Nat.2 * (y * y) implies x = Nat.0 }
    forall(y: Nat) { x * x = Nat.2 * (y * y) implies x = Nat.0 }
}

/// The Diophantine equation `x² = 2y²` has no nonzero natural solution.
theorem nat_no_nontrivial_sq_eq_two_sq(x: Nat, y: Nat) {
    x * x = Nat.2 * (y * y) implies x = Nat.0 and y = Nat.0
} by {
    if x * x = Nat.2 * (y * y) {
        nat_sq_eq_two_sq_zero(x)
        forall(w: Nat) { x * x = Nat.2 * (w * w) implies x = Nat.0 }
        x * x = Nat.2 * (y * y) implies x = Nat.0
        x = Nat.0
        x * x = Nat.2 * (y * y)
        Nat.0 * Nat.0 = Nat.0
        Nat.0 = Nat.2 * (y * y)
        Nat.2 * (y * y) = Nat.0
        mul_to_zero(Nat.2, y * y)
        if Nat.2 = Nat.0 {
            false
        }
        y * y = Nat.0
        mul_to_zero(y, y)
        if y = Nat.0 {
        } else {
            false
        }
        y = Nat.0
        x = Nat.0 and y = Nat.0
    }
}

/// The Diophantine equation `x² = 2y²` has no nonzero integer solution:
/// taking absolute values reduces to the natural-number statement.
theorem int_no_nontrivial_sq_eq_two_sq(x: Int, y: Int) {
    x * x = Int.2 * (y * y) implies x = Int.0 and y = Int.0
} by {
    if x * x = Int.2 * (y * y) {
        abs_mul(x, x)
        abs(x * x) = abs(x) * abs(x)
        abs_mul(Int.2, y * y)
        abs(Int.2 * (y * y)) = abs(Int.2) * abs(y * y)
        abs_mul(y, y)
        abs(y * y) = abs(y) * abs(y)
        abs(Int.2 * (y * y)) = abs(Int.2) * (abs(y) * abs(y))
        abs_from_nat(Nat.2)
        abs(Int.from_nat(Nat.2)) = Nat.2
        Int.from_nat(Nat.2) = Int.2
        abs(Int.2) = Nat.2
        abs(Int.2 * (y * y)) = Nat.2 * (abs(y) * abs(y))
        abs(x * x) = abs(Int.2 * (y * y))
        abs(x) * abs(x) = Nat.2 * (abs(y) * abs(y))
        nat_no_nontrivial_sq_eq_two_sq(abs(x), abs(y))
        abs(x) = Nat.0 and abs(y) = Nat.0
        abs_zero_imp_zero(x)
        abs(x) = Nat.0 implies x = Int.0
        x = Int.0
        abs_zero_imp_zero(y)
        abs(y) = Nat.0 implies y = Int.0
        y = Int.0
        x = Int.0 and y = Int.0
    }
}

/// Clearing the denominator in the square of a fraction: `((a/b)²)·b² = a²`.
theorem div_square_denom_cancel(a: Rat, b: Rat) {
    b != Rat.0 implies ((a / b) * (a / b)) * (b * b) = a * a
} by {
    if b != Rat.0 {
        mul_cancels_div(a, b)
        (a / b) * b = a
        ((a / b) * (a / b)) * (b * b) = (a / b) * ((a / b) * (b * b))
        (a / b) * ((a / b) * (b * b)) = (a / b) * (((a / b) * b) * b)
        (a / b) * (((a / b) * b) * b) = (a / b) * (a * b)
        (a / b) * (a * b) = (a / b) * (b * a)
        (a / b) * (b * a) = ((a / b) * b) * a
        ((a / b) * b) * a = a * a
        ((a / b) * (a / b)) * (b * b) = a * a
    }
}

/// No rational number squares to two.
///
/// This is the classical irrationality of `√2`: writing a rational `r` in
/// lowest terms as `num / denom`, the equality `r² = 2` forces
/// `num² = 2 · denom²` in the integers; the Diophantine equation
/// `x² = 2y²` has no solution with `y != 0` (the infinite-descent argument
/// of `nat_no_nontrivial_sq_eq_two_sq` and its integer analogue), so `r`
/// cannot exist.
theorem no_rat_squares_to_two(r: Rat) {
    r * r = Rat.2 implies false
} by {
    if r * r = Rat.2 {
        div_from_int(r)
        Rat.from_int(r.num) / Rat.from_int(r.denom) = r
        from_int_zero
        Rat.from_int(Int.0) = Rat.0
        if Rat.from_int(r.denom) = Rat.0 {
            Rat.from_int(r.denom) = Rat.from_int(Int.0)
            from_int_cancel(r.denom, Int.0)
            r.denom = Int.0
            denom_nonzero(r)
            r.denom != Int.0
            false
        }
        Rat.from_int(r.denom) != Rat.0
        (Rat.from_int(r.num) / Rat.from_int(r.denom)) *
            (Rat.from_int(r.num) / Rat.from_int(r.denom)) = Rat.2
        ((Rat.from_int(r.num) / Rat.from_int(r.denom)) *
            (Rat.from_int(r.num) / Rat.from_int(r.denom))) *
            (Rat.from_int(r.denom) * Rat.from_int(r.denom)) =
            Rat.2 * (Rat.from_int(r.denom) * Rat.from_int(r.denom))
        div_square_denom_cancel(Rat.from_int(r.num), Rat.from_int(r.denom))
        ((Rat.from_int(r.num) / Rat.from_int(r.denom)) *
            (Rat.from_int(r.num) / Rat.from_int(r.denom))) *
            (Rat.from_int(r.denom) * Rat.from_int(r.denom)) =
            Rat.from_int(r.num) * Rat.from_int(r.num)
        Rat.from_int(r.num) * Rat.from_int(r.num) =
            Rat.2 * (Rat.from_int(r.denom) * Rat.from_int(r.denom))
        mul_int_eq_int_mul(r.num, r.num)
        Rat.from_int(r.num * r.num) = Rat.from_int(r.num) * Rat.from_int(r.num)
        Rat.from_int(r.num * r.num) =
            Rat.2 * (Rat.from_int(r.denom) * Rat.from_int(r.denom))
        mul_int_eq_int_mul(r.denom, r.denom)
        Rat.from_int(r.denom * r.denom) = Rat.from_int(r.denom) * Rat.from_int(r.denom)
        Rat.from_int(r.num * r.num) = Rat.2 * Rat.from_int(r.denom * r.denom)
        mul_int_eq_int_mul(Int.2, r.denom * r.denom)
        Rat.from_int(Int.2 * (r.denom * r.denom)) =
            Rat.from_int(Int.2) * Rat.from_int(r.denom * r.denom)
        Rat.from_int(r.num * r.num) =
            Rat.from_int(Int.2 * (r.denom * r.denom))
        from_int_cancel(r.num * r.num, Int.2 * (r.denom * r.denom))
        r.num * r.num = Int.2 * (r.denom * r.denom)
        int_no_nontrivial_sq_eq_two_sq(r.num, r.denom)
        r.num = Int.0 and r.denom = Int.0
        r.denom = Int.0
        denom_nonzero(r)
        r.denom != Int.0
        false
    }
}

/// `X^2 - 2` has no rational root.
///
/// A quadratic over the rationals is irreducible exactly when it has no
/// rational root, so this is the irreducibility verification behind the
/// statement that the minimal polynomial of `√2` over Q is `X^2 - 2`.
theorem rat_x_squared_minus_two_has_no_root {
    not exists(r: Rat) {
        polynomial_eval(rat_x_squared_minus_two, r) = Rat.0
    }
} by {
    if exists(r: Rat) {
        polynomial_eval(rat_x_squared_minus_two, r) = Rat.0
    } {
        let (r: Rat) satisfy {
            polynomial_eval(rat_x_squared_minus_two, r) = Rat.0
        }
        polynomial_eval(rat_x_squared_minus_two, r) = Rat.0
        rat_x_squared_minus_two_eval(r)
        polynomial_eval(rat_x_squared_minus_two, r) = r * r - Rat.2
        r * r - Rat.2 = Rat.0
        r * r = Rat.2
        no_rat_squares_to_two(r)
        false
    }
}

// The minimal polynomial of √2 over Q is X^2 - 2:
//
//     * X^2 - 2 is monic with rational coefficients;
//     * √2 is a root of X^2 - 2 (section 2);
//     * X^2 - 2 has no rational root (`rat_x_squared_minus_two_has_no_root`),
//       and a quadratic over a field with no root is irreducible, so X^2 - 2
//       is irreducible over Q.
//
// Hence X^2 - 2 is the (unique) monic irreducible polynomial over Q that
// vanishes at √2, i.e. the minimal polynomial of √2 over Q, and
// [Q(√2) : Q] = 2.  A formal definition of "minimal polynomial" (monic,
// irreducible, of least degree among the polynomials vanishing at √2) is not
// yet in the library, so the statement is recorded here as a comment.

// ---------------------------------------------------------------------------
// 4. The automorphism of Q(√2)
// ---------------------------------------------------------------------------
//
// The field Q(√2) is realized inside the complex numbers: the element
// `a + b√2` is `qsqrt2(a, b)` below (the coefficients are kept as reals; for
// the Galois-theoretic reading they range over the rationals, where the
// representation `a + b√2` is unique because √2 is irrational).
//
// The nontrivial automorphism σ of Q(√2)/Q sends √2 to -√2, i.e.
// `a + b√2 ↦ a - b√2` (`qsqrt2_aut`).  We verify for concrete elements that
// σ preserves addition and multiplication.
//
// The library's conjugation machinery (complex conjugation as a field
// homomorphism, `complex_conj_field_hom`) is the template for such maps.
// Complex conjugation fixes every real number, and Q(√2) sits inside the
// reals, so conjugation fixes Q(√2) pointwise: it is the *trivial*
// automorphism of this model.  The nontrivial automorphism σ is the
// conjugate-square-root map `a + b√2 ↦ a - b√2`, which is not an extension
// of complex conjugation but the Galois automorphism swapping the two roots
// of X^2 - 2.

/// An element of Q(√2), written `a + b√2`, embedded in the complex numbers.
define qsqrt2(a: Real, b: Real) -> Complex {
    Complex.from_real(a) + Complex.from_real(b) * Complex.from_real(sqrt_two)
}

/// The nontrivial automorphism σ of Q(√2): `a + b√2 ↦ a - b√2`.
define qsqrt2_aut(a: Real, b: Real) -> Complex {
    qsqrt2(a, -b)
}

/// Complex conjugation fixes Q(√2) pointwise.
///
/// `complex_conj_field_hom` is the library's conjugation machinery; it fixes
/// embedded reals, and `a + b√2` is a sum and product of embedded reals, so
/// it is fixed by conjugation.
theorem qsqrt2_conj_fixed(a: Real, b: Real) {
    complex_conj_field_hom.hom(qsqrt2(a, b)) = qsqrt2(a, b)
} by {
    field_hom_add(complex_conj_field_hom, Complex.from_real(a),
        Complex.from_real(b) * Complex.from_real(sqrt_two))
    complex_conj_field_hom.hom(Complex.from_real(a) + Complex.from_real(b) * Complex.from_real(sqrt_two)) =
        complex_conj_field_hom.hom(Complex.from_real(a)) +
        complex_conj_field_hom.hom(Complex.from_real(b) * Complex.from_real(sqrt_two))
    complex_conj_field_hom_from_real(a)
    complex_conj_field_hom.hom(Complex.from_real(a)) = Complex.from_real(a)
    field_hom_mul(complex_conj_field_hom, Complex.from_real(b), Complex.from_real(sqrt_two))
    complex_conj_field_hom.hom(Complex.from_real(b) * Complex.from_real(sqrt_two)) =
        complex_conj_field_hom.hom(Complex.from_real(b)) *
        complex_conj_field_hom.hom(Complex.from_real(sqrt_two))
    complex_conj_field_hom_from_real(b)
    complex_conj_field_hom.hom(Complex.from_real(b)) = Complex.from_real(b)
    complex_conj_field_hom_from_real(sqrt_two)
    complex_conj_field_hom.hom(Complex.from_real(sqrt_two)) = Complex.from_real(sqrt_two)
    complex_conj_field_hom.hom(Complex.from_real(b) * Complex.from_real(sqrt_two)) =
        Complex.from_real(b) * Complex.from_real(sqrt_two)
    complex_conj_field_hom.hom(qsqrt2(a, b)) = qsqrt2(a, b)
}

/// The automorphism σ sends √2 to -√2.
theorem qsqrt2_aut_sqrt_two {
    qsqrt2_aut(Real.0, Real.1) = qsqrt2(Real.0, -Real.1)
} by {
    qsqrt2_aut(Real.0, Real.1) = qsqrt2(Real.0, -Real.1)
}

/// The automorphism σ fixes the rational (real) part of an element.
theorem qsqrt2_aut_fixes_real_part(a: Real) {
    qsqrt2_aut(a, Real.0) = qsqrt2(a, Real.0)
} by {
    qsqrt2_aut(a, Real.0) = qsqrt2(a, -Real.0)
    -Real.0 = Real.0
    qsqrt2(a, -Real.0) = qsqrt2(a, Real.0)
    qsqrt2_aut(a, Real.0) = qsqrt2(a, Real.0)
}

/// The element `a + b√2` is the complex embedding of the real number `a + b√2`.
theorem qsqrt2_eq_from_real(a: Real, b: Real) {
    qsqrt2(a, b) = Complex.from_real(a + b * sqrt_two)
} by {
    qsqrt2(a, b) = Complex.from_real(a) + Complex.from_real(b) * Complex.from_real(sqrt_two)
    real_mul_lifts(b, sqrt_two)
    Complex.from_real(b * sqrt_two) = Complex.from_real(b) * Complex.from_real(sqrt_two)
    Complex.from_real(a) + Complex.from_real(b) * Complex.from_real(sqrt_two) =
        Complex.from_real(a) + Complex.from_real(b * sqrt_two)
    real_add_lifts(a, b * sqrt_two)
    Complex.from_real(a + b * sqrt_two) =
        Complex.from_real(a) + Complex.from_real(b * sqrt_two)
    Complex.from_real(a) + Complex.from_real(b) * Complex.from_real(sqrt_two) =
        Complex.from_real(a + b * sqrt_two)
    qsqrt2(a, b) = Complex.from_real(a + b * sqrt_two)
}

/// The square of `√2` inside the complex numbers is two.
theorem from_real_sqrt_two_square_complex {
    Complex.from_real(sqrt_two) * Complex.from_real(sqrt_two) = Complex.from_real(two)
} by {
    real_mul_lifts(sqrt_two, sqrt_two)
    Complex.from_real(sqrt_two * sqrt_two) =
        Complex.from_real(sqrt_two) * Complex.from_real(sqrt_two)
    sqrt_two_square
    sqrt_two * sqrt_two = two
    Complex.from_real(two) =
        Complex.from_real(sqrt_two) * Complex.from_real(sqrt_two)
    Complex.from_real(sqrt_two) * Complex.from_real(sqrt_two) = Complex.from_real(two)
}

/// Multiplication of the real forms `a + b√2` and `c + d√2`.
theorem real_qsqrt2_mul_formula(a: Real, b: Real, c: Real, d: Real) {
    (a + b * sqrt_two) * (c + d * sqrt_two) =
        a * c + two * (b * d) + (a * d + b * c) * sqrt_two
} by {
    mul_distrib_left(a, b * sqrt_two, c + d * sqrt_two)
    (a + b * sqrt_two) * (c + d * sqrt_two) =
        a * (c + d * sqrt_two) + (b * sqrt_two) * (c + d * sqrt_two)
    mul_distrib_right(a, c, d * sqrt_two)
    a * (c + d * sqrt_two) = a * c + a * (d * sqrt_two)
    mul_distrib_right(b * sqrt_two, c, d * sqrt_two)
    (b * sqrt_two) * (c + d * sqrt_two) =
        (b * sqrt_two) * c + (b * sqrt_two) * (d * sqrt_two)
    a * (c + d * sqrt_two) + (b * sqrt_two) * (c + d * sqrt_two) =
        a * c + a * (d * sqrt_two) + (b * sqrt_two) * c + (b * sqrt_two) * (d * sqrt_two)
    mul_assoc(a, d, sqrt_two)
    a * (d * sqrt_two) = (a * d) * sqrt_two
    mul_assoc(b, sqrt_two, c)
    b * (sqrt_two * c) = (b * sqrt_two) * c
    real_mul_comm(sqrt_two, c)
    sqrt_two * c = c * sqrt_two
    b * (sqrt_two * c) = b * (c * sqrt_two)
    b * (c * sqrt_two) = (b * c) * sqrt_two
    (b * sqrt_two) * c = (b * c) * sqrt_two
    mul_assoc(b, sqrt_two, d * sqrt_two)
    b * (sqrt_two * (d * sqrt_two)) = (b * sqrt_two) * (d * sqrt_two)
    mul_assoc(sqrt_two, d, sqrt_two)
    sqrt_two * (d * sqrt_two) = (sqrt_two * d) * sqrt_two
    real_mul_comm(sqrt_two, d)
    sqrt_two * d = d * sqrt_two
    sqrt_two * (d * sqrt_two) = (d * sqrt_two) * sqrt_two
    mul_assoc(d, sqrt_two, sqrt_two)
    d * (sqrt_two * sqrt_two) = (d * sqrt_two) * sqrt_two
    mul_assoc(b, d, sqrt_two * sqrt_two)
    b * (d * (sqrt_two * sqrt_two)) = (b * d) * (sqrt_two * sqrt_two)
    b * (sqrt_two * (d * sqrt_two)) = b * (d * (sqrt_two * sqrt_two))
    (b * sqrt_two) * (d * sqrt_two) = (b * d) * (sqrt_two * sqrt_two)
    sqrt_two_square
    sqrt_two * sqrt_two = two
    (b * sqrt_two) * (d * sqrt_two) = (b * d) * two
    real_mul_comm(b * d, two)
    (b * d) * two = two * (b * d)
    (b * sqrt_two) * (d * sqrt_two) = two * (b * d)
    mul_distrib_right(sqrt_two, a * d, b * c)
    sqrt_two * ((a * d) + (b * c)) = sqrt_two * (a * d) + sqrt_two * (b * c)
    real_mul_comm(sqrt_two, a * d)
    sqrt_two * (a * d) = (a * d) * sqrt_two
    real_mul_comm(sqrt_two, b * c)
    sqrt_two * (b * c) = (b * c) * sqrt_two
    (a * d) * sqrt_two + (b * c) * sqrt_two = sqrt_two * (a * d + b * c)
    real_mul_comm(a * d + b * c, sqrt_two)
    (a * d + b * c) * sqrt_two = sqrt_two * (a * d + b * c)
    (a * d) * sqrt_two + (b * c) * sqrt_two = (a * d + b * c) * sqrt_two
    a * c + a * (d * sqrt_two) + (b * sqrt_two) * c + (b * sqrt_two) * (d * sqrt_two) =
        a * c + (a * d) * sqrt_two + (b * c) * sqrt_two + two * (b * d)
    a * c + (a * d) * sqrt_two + (b * c) * sqrt_two + two * (b * d) =
        a * c + ((a * d) * sqrt_two + (b * c) * sqrt_two) + two * (b * d)
    a * c + ((a * d) * sqrt_two + (b * c) * sqrt_two) + two * (b * d) =
        a * c + (a * d + b * c) * sqrt_two + two * (b * d)
    a * c + ((a * d + b * c) * sqrt_two + two * (b * d)) =
        (a * c + (a * d + b * c) * sqrt_two) + two * (b * d)
    a * c + (two * (b * d) + (a * d + b * c) * sqrt_two) =
        a * c + ((a * d + b * c) * sqrt_two + two * (b * d))
    (a * c + two * (b * d)) + (a * d + b * c) * sqrt_two =
        a * c + (two * (b * d) + (a * d + b * c) * sqrt_two)
    a * c + two * (b * d) + (a * d + b * c) * sqrt_two =
        (a * c + two * (b * d)) + (a * d + b * c) * sqrt_two
    (a + b * sqrt_two) * (c + d * sqrt_two) =
        a * c + two * (b * d) + (a * d + b * c) * sqrt_two
}

/// Addition in Q(√2): `(a + b√2) + (c + d√2) = (a + c) + (b + d)√2`.
theorem qsqrt2_add(a: Real, b: Real, c: Real, d: Real) {
    qsqrt2(a, b) + qsqrt2(c, d) = qsqrt2(a + c, b + d)
} by {
    qsqrt2_eq_from_real(a, b)
    qsqrt2(a, b) = Complex.from_real(a + b * sqrt_two)
    qsqrt2_eq_from_real(c, d)
    qsqrt2(c, d) = Complex.from_real(c + d * sqrt_two)
    qsqrt2(a, b) + qsqrt2(c, d) =
        Complex.from_real(a + b * sqrt_two) + Complex.from_real(c + d * sqrt_two)
    real_add_lifts(a + b * sqrt_two, c + d * sqrt_two)
    Complex.from_real((a + b * sqrt_two) + (c + d * sqrt_two)) =
        Complex.from_real(a + b * sqrt_two) + Complex.from_real(c + d * sqrt_two)
    (a + b * sqrt_two) + (c + d * sqrt_two) = a + (b * sqrt_two + (c + d * sqrt_two))
    a + (b * sqrt_two + (c + d * sqrt_two)) = a + ((b * sqrt_two + c) + d * sqrt_two)
    a + ((b * sqrt_two + c) + d * sqrt_two) = a + ((c + b * sqrt_two) + d * sqrt_two)
    a + ((c + b * sqrt_two) + d * sqrt_two) = a + (c + (b * sqrt_two + d * sqrt_two))
    a + (c + (b * sqrt_two + d * sqrt_two)) = (a + c) + (b * sqrt_two + d * sqrt_two)
    mul_distrib_right(sqrt_two, b, d)
    sqrt_two * (b + d) = sqrt_two * b + sqrt_two * d
    real_mul_comm(sqrt_two, b)
    sqrt_two * b = b * sqrt_two
    real_mul_comm(sqrt_two, d)
    sqrt_two * d = d * sqrt_two
    real_mul_comm(b + d, sqrt_two)
    (b + d) * sqrt_two = sqrt_two * (b + d)
    b * sqrt_two + d * sqrt_two = (b + d) * sqrt_two
    (a + c) + (b * sqrt_two + d * sqrt_two) = (a + c) + (b + d) * sqrt_two
    (a + b * sqrt_two) + (c + d * sqrt_two) = (a + c) + (b + d) * sqrt_two
    Complex.from_real((a + c) + (b + d) * sqrt_two) =
        Complex.from_real((a + b * sqrt_two) + (c + d * sqrt_two))
    qsqrt2_eq_from_real(a + c, b + d)
    qsqrt2(a + c, b + d) = Complex.from_real((a + c) + (b + d) * sqrt_two)
    qsqrt2(a, b) + qsqrt2(c, d) = qsqrt2(a + c, b + d)
}

/// Multiplication in Q(√2):
/// `(a + b√2)(c + d√2) = (ac + 2bd) + (ad + bc)√2`.
theorem qsqrt2_mul(a: Real, b: Real, c: Real, d: Real) {
    qsqrt2(a, b) * qsqrt2(c, d) = qsqrt2(a * c + two * (b * d), a * d + b * c)
} by {
    qsqrt2_eq_from_real(a, b)
    qsqrt2(a, b) = Complex.from_real(a + b * sqrt_two)
    qsqrt2_eq_from_real(c, d)
    qsqrt2(c, d) = Complex.from_real(c + d * sqrt_two)
    qsqrt2(a, b) * qsqrt2(c, d) =
        Complex.from_real(a + b * sqrt_two) * Complex.from_real(c + d * sqrt_two)
    real_mul_lifts(a + b * sqrt_two, c + d * sqrt_two)
    Complex.from_real((a + b * sqrt_two) * (c + d * sqrt_two)) =
        Complex.from_real(a + b * sqrt_two) * Complex.from_real(c + d * sqrt_two)
    real_qsqrt2_mul_formula(a, b, c, d)
    (a + b * sqrt_two) * (c + d * sqrt_two) =
        a * c + two * (b * d) + (a * d + b * c) * sqrt_two
    Complex.from_real(a * c + two * (b * d) + (a * d + b * c) * sqrt_two) =
        Complex.from_real((a + b * sqrt_two) * (c + d * sqrt_two))
    qsqrt2_eq_from_real(a * c + two * (b * d), a * d + b * c)
    qsqrt2(a * c + two * (b * d), a * d + b * c) =
        Complex.from_real(a * c + two * (b * d) + (a * d + b * c) * sqrt_two)
    qsqrt2(a, b) * qsqrt2(c, d) = qsqrt2(a * c + two * (b * d), a * d + b * c)
}

/// The automorphism σ preserves addition: `σ(x + y) = σ(x) + σ(y)`.
theorem qsqrt2_aut_add(a: Real, b: Real, c: Real, d: Real) {
    qsqrt2(a + c, -(b + d)) = qsqrt2(a, -b) + qsqrt2(c, -d)
} by {
    qsqrt2_add(a, -b, c, -d)
    qsqrt2(a, -b) + qsqrt2(c, -d) = qsqrt2(a + c, -b + -d)
    neg_distrib(b, d)
    -(b + d) = -b + -d
    -b + -d = -(b + d)
    qsqrt2(a, -b) + qsqrt2(c, -d) = qsqrt2(a + c, -(b + d))
    qsqrt2(a + c, -(b + d)) = qsqrt2(a, -b) + qsqrt2(c, -d)
}

/// The automorphism σ preserves multiplication: `σ(x·y) = σ(x)·σ(y)`.
theorem qsqrt2_aut_mul(a: Real, b: Real, c: Real, d: Real) {
    qsqrt2(a * c + two * (b * d), -(a * d + b * c)) = qsqrt2(a, -b) * qsqrt2(c, -d)
} by {
    qsqrt2_mul(a, -b, c, -d)
    qsqrt2(a, -b) * qsqrt2(c, -d) =
        qsqrt2(a * c + two * ((-b) * (-d)), a * (-d) + (-b) * c)
    mul_neg_neg[Real](b, d)
    (-b) * (-d) = b * d
    two * ((-b) * (-d)) = two * (b * d)
    a * c + two * ((-b) * (-d)) = a * c + two * (b * d)
    mul_neg_right(a, d)
    a * (-d) = -(a * d)
    mul_neg_left(b, c)
    (-b) * c = -(b * c)
    a * (-d) + (-b) * c = -(a * d) + -(b * c)
    neg_distrib(a * d, b * c)
    -(a * d + b * c) = -(a * d) + -(b * c)
    -(a * d) + -(b * c) = -(a * d + b * c)
    a * (-d) + (-b) * c = -(a * d + b * c)
    qsqrt2(a, -b) * qsqrt2(c, -d) =
        qsqrt2(a * c + two * (b * d), -(a * d + b * c))
    qsqrt2(a * c + two * (b * d), -(a * d + b * c)) = qsqrt2(a, -b) * qsqrt2(c, -d)
}

// ---------------------------------------------------------------------------
// 5. The fundamental theorem of Galois theory (statement)
// ---------------------------------------------------------------------------
//
// For a finite Galois extension E/F with Galois group G = Aut(E/F), the
// fundamental theorem of Galois theory states:
//
//     * the map H ↦ E^H (the fixed field of the subgroup H) is a bijection
//       between the subgroups of G and the intermediate fields F ⊆ K ⊆ E;
//     * the correspondence is inclusion-reversing: H1 ⊆ H2 iff E^H2 ⊆ E^H1;
//     * [E : E^H] = |H| and [E^H : F] = [G : H] (the degree formulas);
//     * H is normal in G iff E^H is a Galois extension of F, and then
//       G / H ≅ Aut(E^H / F) (the quotient correspondence).
//
// For the quadratic example Q(√2)/Q of this file, the Galois group is
// Aut(Q(√2)/Q) = {id, σ} with σ : √2 ↦ -√2 (section 4), and the
// correspondence has exactly two intermediate fields: Q and Q(√2) itself,
// matched with the subgroups {id} and {id, σ} of the two-element group.
//
// A full formal statement needs the library's subfield and automorphism-group
// machinery together with a formal definition of "Galois extension" (normal
// and separable), none of which is yet in the library; the theorem is
// therefore recorded here as a comment only.


