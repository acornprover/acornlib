from algebra.semigroup import Semigroup

/// Extending the multiplicative semigroup with commutativity.
typeclass S: CommSemigroup extends Semigroup {
    /// The multiplication operation must be commutative: `a * b = b * a`.
    commutative(a: S, b: S) {
        a * b = b * a
    }
}