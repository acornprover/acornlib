// Group theory deepening: Lagrange's theorem, the order of an element,
// prime-order groups are cyclic, the cyclic group C_4, and the center of a group.

from finite_group import FiniteGroup, FiniteSubgroup, cyclic_subgroup_of,
    finite_subgroup_order_divides_group_order, finite_cyclic_subgroup_order_divides_group_order,
    finite_subgroup_contains_eq, finite_subgroup_as_set_contains_finite_subgroup_eq,
    cyclic_subgroup_order_pos, pow_cyclic_subgroup_order_eq_identity,
    cyclic_subgroup_contains_power, cyclic_subgroup_elements_eq, pow_mod_cyclic_subgroup_order
from algebra.group import Group, inverse_left, right_cancel
from algebra.subgroup import subgroup_constraint, identity_constraint, closure_constraint,
    inverse_constraint
from nat import Nat, divides_symm, divides_self, divides_gcd, gcd_comm, gcd_divides_left,
    gcd_of_prime, trichotomy, lt_suc_right, lt_imp_lte_suc, pos_of_ne_zero
from number_theory import mod_lt
from list import List, map, unique_contains_imp_contains, map_contains, lt_of_range_contains
from data.basic.set import Set, set_ext, singleton_contains_eq, list_set, list_set_contains_eq,
    unique_list_set_cardinality_is_length, set_eq_universal_of_forall_contains,
    cardinality_is_well_defined, singleton_set_cardinality_is_one
numerals Nat

/// Lagrange's theorem: the order of a finite subgroup divides the order of the group.
theorem lagrange_theorem[G: FiniteGroup](s: FiniteSubgroup[G]) {
    s.order.divides(G.order)
} by {
    finite_subgroup_order_divides_group_order(s)
    s.order.divides(G.order)
}

/// The order of an element (the size of its cyclic subgroup) divides the group order.
theorem element_order_divides_group_order[G: FiniteGroup](g: G) {
    cyclic_subgroup_of(g).order.divides(G.order)
} by {
    finite_cyclic_subgroup_order_divides_group_order(g)
    cyclic_subgroup_of(g).order.divides(G.order)
}

/// Corollary of Lagrange's theorem: applying Lagrange to the cyclic subgroup
/// generated by an element shows that its order divides the group order.
theorem element_order_divides_group_order_of_lagrange[G: FiniteGroup](g: G) {
    cyclic_subgroup_of(g).order.divides(G.order)
} by {
    lagrange_theorem(cyclic_subgroup_of(g))
    cyclic_subgroup_of(g).order.divides(G.order)
}

/// True when an element generates the whole group (its cyclic subgroup has full order).
define is_generator[G: FiniteGroup](g: G) -> Bool {
    cyclic_subgroup_of(g).order = G.order
}

/// True when the group is cyclic: some element generates it.
let is_cyclic[G: FiniteGroup] = exists(g: G) {
    cyclic_subgroup_of(g).order = G.order
}

/// The universal set of a finite group has cardinality equal to the group order.
theorem finite_group_universal_set_cardinality_is_order[G: FiniteGroup] {
    Set[G].universal_set.cardinality_is(G.order)
} by {
    forall(x: G) {
        G.elements.contains(x)
        list_set_contains_eq[G](G.elements, x)
        list_set[G](G.elements).contains(x)
    }
    set_eq_universal_of_forall_contains[G](list_set[G](G.elements))
    list_set[G](G.elements) = Set[G].universal_set
    G.elements.is_unique
    unique_list_set_cardinality_is_length[G](G.elements)
    list_set[G](G.elements).cardinality_is(G.elements.length)
    G.order = G.elements.length
    Set[G].universal_set.cardinality_is(G.order)
}

/// If every element of the group is the identity, the universal set is the singleton identity.
theorem universal_set_eq_singleton_of_all_identity[G: FiniteGroup] {
    (forall(g: G) { g = G.1 }) implies Set[G].universal_set = Set[G].singleton(G.1)
} by {
    if forall(g: G) { g = G.1 } {
        forall(x: G) {
            Set[G].universal_set.contains(x)
            x = G.1
            singleton_contains_eq[G](G.1, x)
            Set[G].singleton(G.1).contains(x)
        }
        set_ext[G](Set[G].universal_set, Set[G].singleton(G.1))
        Set[G].universal_set = Set[G].singleton(G.1)
    }
}

/// A finite group with more than one element has a non-identity element.
theorem exists_non_identity_of_order_gt_one[G: FiniteGroup] {
    G.order > Nat.1 implies exists(g: G) { g != G.1 }
} by {
    if G.order > Nat.1 {
        if exists(g: G) { g != G.1 } {
            exists(g: G) { g != G.1 }
        }
        if not (exists(g: G) { g != G.1 }) {
            forall(g: G) {
                if g = G.1 {
                    g = G.1
                } else {
                    g != G.1
                    exists(h: G) { h = g and h != G.1 }
                    exists(h: G) { h != G.1 }
                    false
                }
                g = G.1
            }
            universal_set_eq_singleton_of_all_identity[G]
            Set[G].universal_set = Set[G].singleton(G.1)
            finite_group_universal_set_cardinality_is_order[G]
            Set[G].universal_set.cardinality_is(G.order)
            singleton_set_cardinality_is_one[G](G.1)
            Set[G].singleton(G.1).cardinality_is(Nat.1)
            cardinality_is_well_defined[G](Set[G].universal_set, G.order, Nat.1)
            G.order = Nat.1
            false
        }
        exists(g: G) { g != G.1 }
    }
}

/// A number that divides another number equals their greatest common divisor.
theorem divides_gcd_self(a: Nat, b: Nat) {
    a.divides(b) implies a.gcd(b) = a
} by {
    if a.divides(b) {
        divides_self(a)
        divides_gcd(a, a, b)
        a.divides(a.gcd(b))
        gcd_divides_left(a, b)
        a.gcd(b).divides(a)
        divides_symm(a.gcd(b), a)
        a.gcd(b) = a
    }
}

/// A divisor of a prime that is greater than one must be the prime itself.
theorem prime_divisor_gt_one_eq_self(p: Nat, d: Nat) {
    p.is_prime and d.divides(p) and d > Nat.1 implies d = p
} by {
    if p.is_prime and d.divides(p) and d > Nat.1 {
        gcd_of_prime(p, d)
        if p.gcd(d) = Nat.1 {
            divides_gcd_self(d, p)
            d.gcd(p) = d
            gcd_comm(d, p)
            d.gcd(p) = p.gcd(d)
            p.gcd(d) = Nat.1
            d = Nat.1
            false
        }
        p.divides(d)
        divides_symm(p, d)
        p = d
        d = p
    }
}

/// The cyclic subgroup generated by a non-identity element has order greater than one.
theorem cyclic_subgroup_order_gt_one_of_non_identity[G: FiniteGroup](g: G) {
    g != G.1 implies cyclic_subgroup_of(g).order > Nat.1
} by {
    if g != G.1 {
        cyclic_subgroup_order_pos(g)
        cyclic_subgroup_of(g).order > Nat.0
        if cyclic_subgroup_of(g).order = Nat.1 {
            pow_cyclic_subgroup_order_eq_identity(g)
            g.pow(cyclic_subgroup_of(g).order) = G.1
            g.pow(Nat.1) = G.1
            g.pow(Nat.1) = g
            g = G.1
            false
        }
        cyclic_subgroup_of(g).order != Nat.1
        pos_of_ne_zero(cyclic_subgroup_of(g).order)
        Nat.0 < cyclic_subgroup_of(g).order
        lt_imp_lte_suc(Nat.0, cyclic_subgroup_of(g).order)
        Nat.1 <= cyclic_subgroup_of(g).order
        trichotomy(Nat.1, cyclic_subgroup_of(g).order)
        if cyclic_subgroup_of(g).order < Nat.1 {
            lt_suc_right(cyclic_subgroup_of(g).order, Nat.0)
            cyclic_subgroup_of(g).order = Nat.0
            false
        }
        if Nat.1 = cyclic_subgroup_of(g).order {
            false
        }
        Nat.1 < cyclic_subgroup_of(g).order
        cyclic_subgroup_of(g).order > Nat.1
    }
}

/// A group of prime order is cyclic: a non-identity element generates the whole group.
theorem prime_order_group_is_cyclic[G: FiniteGroup] {
    G.order.is_prime implies exists(g: G) {
        cyclic_subgroup_of(g).order = G.order
    }
} by {
    if G.order.is_prime {
        G.order > Nat.1
        exists_non_identity_of_order_gt_one[G]
        let g: G satisfy { g != G.1 }
        cyclic_subgroup_order_gt_one_of_non_identity(g)
        cyclic_subgroup_of(g).order > Nat.1
        finite_cyclic_subgroup_order_divides_group_order(g)
        cyclic_subgroup_of(g).order.divides(G.order)
        prime_divisor_gt_one_eq_self(G.order, cyclic_subgroup_of(g).order)
        cyclic_subgroup_of(g).order = G.order
        exists(h: G) { cyclic_subgroup_of(h).order = G.order }
    }
}

/// Membership in the cyclic subgroup of an element is being a power below its order.
theorem cyclic_subgroup_contains_iff_power_lt_order[G: FiniteGroup](g: G, x: G) {
    cyclic_subgroup_of(g).contains(x) iff exists(k: Nat) {
        k < cyclic_subgroup_of(g).order and x = g.pow(k)
    }
} by {
    cyclic_subgroup_order_pos(g)
    if cyclic_subgroup_of(g).contains(x) {
        cyclic_subgroup_elements_eq(g)
        cyclic_subgroup_of(g).elements = map(G.order.range, g.pow).unique
        finite_subgroup_contains_eq(cyclic_subgroup_of(g), x)
        cyclic_subgroup_of(g).elements.contains(x)
        map(G.order.range, g.pow).unique.contains(x)
        unique_contains_imp_contains(map(G.order.range, g.pow), x)
        map(G.order.range, g.pow).contains(x)
        map_contains(G.order.range, g.pow, x)
        let k: Nat satisfy { G.order.range.contains(k) and g.pow(k) = x }
        lt_of_range_contains(G.order, k)
        k < G.order
        cyclic_subgroup_of(g).order != Nat.0
        mod_lt(k, cyclic_subgroup_of(g).order)
        k.mod(cyclic_subgroup_of(g).order) < cyclic_subgroup_of(g).order
        pow_mod_cyclic_subgroup_order(g, k)
        g.pow(k.mod(cyclic_subgroup_of(g).order)) = g.pow(k)
        g.pow(k.mod(cyclic_subgroup_of(g).order)) = x
        exists(m: Nat) { m = k.mod(cyclic_subgroup_of(g).order) and m < cyclic_subgroup_of(g).order and x = g.pow(m) }
        exists(m: Nat) { m < cyclic_subgroup_of(g).order and x = g.pow(m) }
    }
    if exists(k: Nat) { k < cyclic_subgroup_of(g).order and x = g.pow(k) } {
        let k: Nat satisfy { k < cyclic_subgroup_of(g).order and x = g.pow(k) }
        cyclic_subgroup_contains_power(g, k)
        cyclic_subgroup_of(g).contains(g.pow(k))
        cyclic_subgroup_of(g).contains(x)
    }
    cyclic_subgroup_of(g).contains(x) iff exists(k: Nat) {
        k < cyclic_subgroup_of(g).order and x = g.pow(k)
    }
}

/// True when an element is one of the four elements e, a, a^2, a^3 of a cyclic group of order four.
define c4_elem[G: Group](g: G, x: G) -> Bool {
    x = G.1 or x = g or x = g.pow(2) or x = g.pow(3)
}

/// Membership in the finite set of the four C_4 elements.
theorem c4_elem_eq_disjunction[G: Group](g: G, x: G) {
    c4_elem(g, x) = (x = G.1 or x = g or x = g.pow(2) or x = g.pow(3))
}

/// The identity is one of the four C_4 elements.
theorem c4_or_intro_0[G: Group](g: G, x: G) {
    x = G.1 implies c4_elem(g, x)
}
/// The generator is one of the four C_4 elements.
theorem c4_or_intro_1[G: Group](g: G, x: G) {
    x = g implies c4_elem(g, x)
}
/// The square of the generator is one of the four C_4 elements.
theorem c4_or_intro_2[G: Group](g: G, x: G) {
    x = g.pow(2) implies c4_elem(g, x)
}
/// The cube of the generator is one of the four C_4 elements.
theorem c4_or_intro_3[G: Group](g: G, x: G) {
    x = g.pow(3) implies c4_elem(g, x)
}

/// A natural number below four is zero, one, two, or three.
theorem lt_four_cases(k: Nat) {
    k < 4 implies (k = Nat.0 or k = Nat.1 or k = Nat.2 or k = Nat.3)
} by {
    if k < 4 {
        lt_suc_right(k, Nat.3)
        if k = Nat.3 {
            k = Nat.0 or k = Nat.1 or k = Nat.2 or k = Nat.3
        } else {
            k < Nat.3
            lt_suc_right(k, Nat.2)
            if k = Nat.2 {
                k = Nat.0 or k = Nat.1 or k = Nat.2 or k = Nat.3
            } else {
                k < Nat.2
                lt_suc_right(k, Nat.1)
                if k = Nat.1 {
                    k = Nat.0 or k = Nat.1 or k = Nat.2 or k = Nat.3
                } else {
                    k < Nat.1
                    lt_suc_right(k, Nat.0)
                    if k = Nat.0 {
                        k = Nat.0 or k = Nat.1 or k = Nat.2 or k = Nat.3
                    } else {
                        k < Nat.0
                        false
                    }
                }
            }
        }
    }
}

/// Every member of a cyclic subgroup of order four is one of e, a, a^2, a^3.
theorem c4_forward[G: FiniteGroup](g: G, x: G) {
    cyclic_subgroup_of(g).order = 4 and cyclic_subgroup_of(g).contains(x) implies c4_elem(g, x)
} by {
    if cyclic_subgroup_of(g).order = 4 and cyclic_subgroup_of(g).contains(x) {
        cyclic_subgroup_contains_iff_power_lt_order(g, x)
        cyclic_subgroup_of(g).contains(x)
        exists(k: Nat) { k < cyclic_subgroup_of(g).order and x = g.pow(k) }
        let k: Nat satisfy { k < cyclic_subgroup_of(g).order and x = g.pow(k) }
        x = g.pow(k)
        k < 4
        lt_four_cases(k)
        if k = Nat.0 {
            g.pow(k) = g.pow(Nat.0)
            g.pow(Nat.0) = G.1
            x = G.1
            c4_or_intro_0(g, x)
            c4_elem(g, x)
        }
        if k = Nat.1 {
            g.pow(k) = g.pow(Nat.1)
            g.pow(Nat.1) = g
            x = g
            c4_or_intro_1(g, x)
            c4_elem(g, x)
        }
        if k = Nat.2 {
            x = g.pow(2)
            c4_or_intro_2(g, x)
            c4_elem(g, x)
        }
        if k = Nat.3 {
            x = g.pow(3)
            c4_or_intro_3(g, x)
            c4_elem(g, x)
        }
        c4_elem(g, x)
    }
}

/// Each of e, a, a^2, a^3 belongs to a cyclic subgroup of order four.
theorem c4_backward[G: FiniteGroup](g: G, x: G) {
    cyclic_subgroup_of(g).order = 4 and c4_elem(g, x) implies cyclic_subgroup_of(g).contains(x)
} by {
    if cyclic_subgroup_of(g).order = 4 and c4_elem(g, x) {
        c4_elem_eq_disjunction(g, x)
        c4_elem(g, x) = (x = G.1 or x = g or x = g.pow(2) or x = g.pow(3))
        if x = G.1 {
            g.pow(Nat.0) = G.1
            x = g.pow(Nat.0)
            cyclic_subgroup_contains_power(g, Nat.0)
            cyclic_subgroup_of(g).contains(g.pow(Nat.0))
            cyclic_subgroup_of(g).contains(x)
        }
        if x = g {
            g.pow(Nat.1) = g
            x = g.pow(Nat.1)
            cyclic_subgroup_contains_power(g, Nat.1)
            cyclic_subgroup_of(g).contains(g.pow(Nat.1))
            cyclic_subgroup_of(g).contains(x)
        }
        if x = g.pow(2) {
            cyclic_subgroup_contains_power(g, Nat.2)
            cyclic_subgroup_of(g).contains(g.pow(2))
            cyclic_subgroup_of(g).contains(x)
        }
        if x = g.pow(3) {
            cyclic_subgroup_contains_power(g, Nat.3)
            cyclic_subgroup_of(g).contains(g.pow(3))
            cyclic_subgroup_of(g).contains(x)
        }
        cyclic_subgroup_of(g).contains(x)
    }
}

/// The members of a cyclic subgroup of order four are exactly e, a, a^2, a^3.
theorem c4_membership[G: FiniteGroup](g: G, x: G) {
    cyclic_subgroup_of(g).order = 4 implies (cyclic_subgroup_of(g).contains(x) = c4_elem(g, x))
} by {
    if cyclic_subgroup_of(g).order = 4 {
        if cyclic_subgroup_of(g).contains(x) {
            c4_forward(g, x)
            c4_elem(g, x)
        }
        if c4_elem(g, x) {
            c4_backward(g, x)
            cyclic_subgroup_of(g).contains(x)
        }
        cyclic_subgroup_of(g).contains(x) = c4_elem(g, x)
    }
}

/// The set of elements of a cyclic subgroup of order four is exactly {e, a, a^2, a^3}.
theorem c4_as_set[G: FiniteGroup](g: G) {
    cyclic_subgroup_of(g).order = 4 implies
        cyclic_subgroup_of(g).as_set = Set[G].new(c4_elem[G](g))
} by {
    if cyclic_subgroup_of(g).order = 4 {
        forall(x: G) {
            finite_subgroup_as_set_contains_finite_subgroup_eq(cyclic_subgroup_of(g), x)
            cyclic_subgroup_of(g).as_set.contains(x) = cyclic_subgroup_of(g).contains(x)
            c4_membership(g, x)
            cyclic_subgroup_of(g).contains(x) = c4_elem(g, x)
            Set[G].new(c4_elem[G](g)).contains(x) = c4_elem(g, x)
            cyclic_subgroup_of(g).as_set.contains(x) = Set[G].new(c4_elem[G](g)).contains(x)
        }
        set_ext[G](cyclic_subgroup_of(g).as_set, Set[G].new(c4_elem[G](g)))
        cyclic_subgroup_of(g).as_set = Set[G].new(c4_elem[G](g))
    }
}

/// In a cyclic subgroup of order four, the fourth power of the generator is the identity.
theorem c4_generator_pow_four[G: FiniteGroup](g: G) {
    cyclic_subgroup_of(g).order = 4 implies g.pow(4) = G.1
} by {
    if cyclic_subgroup_of(g).order = 4 {
        pow_cyclic_subgroup_order_eq_identity(g)
        g.pow(cyclic_subgroup_of(g).order) = G.1
        g.pow(4) = G.1
    }
}

/// In a cyclic subgroup of order four, each of e, a, a^2, a^3 is a member.
theorem c4_elements_are_members[G: FiniteGroup](g: G) {
    cyclic_subgroup_of(g).order = 4 implies
        cyclic_subgroup_of(g).contains(G.1) and
        cyclic_subgroup_of(g).contains(g) and
        cyclic_subgroup_of(g).contains(g.pow(2)) and
        cyclic_subgroup_of(g).contains(g.pow(3))
} by {
    if cyclic_subgroup_of(g).order = 4 {
        cyclic_subgroup_contains_power(g, Nat.0)
        cyclic_subgroup_of(g).contains(G.1)
        cyclic_subgroup_contains_power(g, Nat.1)
        cyclic_subgroup_of(g).contains(g)
        cyclic_subgroup_contains_power(g, Nat.2)
        cyclic_subgroup_of(g).contains(g.pow(2))
        cyclic_subgroup_contains_power(g, Nat.3)
        cyclic_subgroup_of(g).contains(g.pow(3))
    }
}

/// True when an element of a group commutes with every element.
define is_central_element[G: Group](x: G) -> Bool {
    forall(g: G) {
        g * x = x * g
    }
}

/// The identity element is central.
theorem center_contains_identity[G: Group] {
    is_central_element[G](G.1)
} by {
    forall(g: G) {
        g * G.1 = g
        G.1 * g = g
        g * G.1 = G.1 * g
    }
}

/// The center is closed under the group operation: the product of central elements is central.
theorem center_closed_under_mul[G: Group](x: G, y: G) {
    is_central_element(x) and is_central_element(y) implies is_central_element(x * y)
} by {
    if is_central_element(x) and is_central_element(y) {
        is_central_element(x) = forall(h: G) { h * x = x * h }
        is_central_element(y) = forall(h: G) { h * y = y * h }
        forall(g: G) {
            g * x = x * g
            g * y = y * g
            g * (x * y) = (g * x) * y
            (g * x) * y = (x * g) * y
            (x * g) * y = x * (g * y)
            x * (g * y) = x * (y * g)
            x * (y * g) = (x * y) * g
            g * (x * y) = (x * y) * g
        }
    }
}

/// The center is closed under inverses: the inverse of a central element is central.
theorem center_closed_under_inverse[G: Group](x: G) {
    is_central_element(x) implies is_central_element(x.inverse)
} by {
    if is_central_element(x) {
        is_central_element(x) = forall(h: G) { h * x = x * h }
        forall(g: G) {
            g * x = x * g
            x.inverse * (g * x) = x.inverse * (x * g)
            x.inverse * (x * g) = (x.inverse * x) * g
            inverse_left(x)
            x.inverse * x = G.1
            (x.inverse * x) * g = G.1 * g
            G.1 * g = g
            x.inverse * (g * x) = g
            (x.inverse * g) * x = x.inverse * (g * x)
            (x.inverse * g) * x = g
            (g * x.inverse) * x = g * (x.inverse * x)
            (g * x.inverse) * x = g * G.1
            g * G.1 = g
            (g * x.inverse) * x = g
            (x.inverse * g) * x = (g * x.inverse) * x
            right_cancel(x, x.inverse * g, g * x.inverse)
            x.inverse * g = g * x.inverse
            g * x.inverse = x.inverse * g
        }
        is_central_element(x.inverse)
    }
}

/// The center of a group is a subgroup: it contains the identity, and is closed under
/// the group operation and under inverses.
theorem center_is_subgroup[G: Group] {
    subgroup_constraint(is_central_element[G])
} by {
    center_contains_identity[G]
    identity_constraint(is_central_element[G])
    forall(x: G, y: G) {
        if is_central_element(x) and is_central_element(y) {
            center_closed_under_mul(x, y)
        }
    }
    closure_constraint(is_central_element[G])
    forall(x: G) {
        if is_central_element(x) {
            center_closed_under_inverse(x)
        }
    }
    inverse_constraint(is_central_element[G])
}
