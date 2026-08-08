from algebra.monoid.monoid import Monoid, MonoidHom, monoid_hom_one, monoid_hom_mul,
    identity_monoid_hom, identity_monoid_hom_hom, compose_monoid_hom, compose_monoid_hom_hom
from data.basic.functions import identity_fn, compose
from algebra.ring.ring import Ring
from algebra.field.field import Field, unique_inverse
from nat import Nat, alt_induction

/// True if an element of a monoid has a two-sided multiplicative inverse.
define is_monoid_unit[M: Monoid](a: M) -> Bool {
    exists(b: M) {
        a * b = M.1 and b * a = M.1
    }
}

/// A unit of a monoid is an element with a two-sided multiplicative inverse.
structure Unit[M: Monoid] {
    /// The element of the monoid.
    val: M
    /// A two-sided inverse of the element.
    inv: M
} constraint {
    val * inv = M.1 and inv * val = M.1
}

/// The value and inverse determine a unit.
theorem unit_ext[M: Monoid](u: Unit[M], v: Unit[M]) {
    u.val = v.val and u.inv = v.inv implies u = v
} by {
    if u.val = v.val and u.inv = v.inv {
        u.val = v.val
        u.inv = v.inv
    }
}

attributes Unit[M: Monoid] {
    /// Unit extensionality from equality of the value and inverse.
    let ext = unit_ext[M]
}

/// The value of a unit multiplied by its inverse is the identity.
theorem unit_val_mul_inv[M: Monoid](u: Unit[M]) {
    u.val * u.inv = M.1
}

/// The inverse of a unit multiplied by its value is the identity.
theorem unit_inv_mul_val[M: Monoid](u: Unit[M]) {
    u.inv * u.val = M.1
}

/// The value of a unit is a unit of the monoid.
theorem unit_val_is_monoid_unit[M: Monoid](u: Unit[M]) {
    is_monoid_unit(u.val)
} by {
    exists(b: M) {
        b = u.inv and u.val * b = M.1 and b * u.val = M.1
    }
}

/// The monoid identity is a unit.
theorem one_is_monoid_unit[M: Monoid] {
    is_monoid_unit(M.1)
} by {
    M.1 * M.1 = M.1
    exists(b: M) {
        b = M.1 and M.1 * b = M.1 and b * M.1 = M.1
    }
}

/// An element with a two-sided multiplicative inverse is a monoid unit.
theorem is_monoid_unit_of_inverse[M: Monoid](a: M, b: M) {
    a * b = M.1 and b * a = M.1 implies is_monoid_unit(a)
} by {
    if a * b = M.1 and b * a = M.1 {
        exists(c: M) {
            c = b and a * c = M.1 and c * a = M.1
        }
    }
}

/// The product of two monoid units is a monoid unit.
theorem monoid_unit_mul_right_inverse[M: Monoid](a: M, b: M, a_inv: M, b_inv: M) {
    a * a_inv = M.1 and a_inv * a = M.1 and b * b_inv = M.1 and b_inv * b = M.1
        implies (a * b) * (b_inv * a_inv) = M.1
} by {
    if a * a_inv = M.1 and a_inv * a = M.1 and b * b_inv = M.1 and b_inv * b = M.1 {
        let c: M = b_inv * a_inv
        (a * b) * c = (a * b) * (b_inv * a_inv)
        (a * b) * (b_inv * a_inv) = a * (b * (b_inv * a_inv))
        b * (b_inv * a_inv) = (b * b_inv) * a_inv
        b * b_inv = M.1
        (b * b_inv) * a_inv = M.1 * a_inv
        M.1 * a_inv = a_inv
        b * (b_inv * a_inv) = a_inv
        a * (b * (b_inv * a_inv)) = a * a_inv
        a * a_inv = M.1
        (a * b) * c = M.1
    }
}

/// The reverse product of inverse witnesses is a left inverse for the product.
theorem monoid_unit_mul_left_inverse[M: Monoid](a: M, b: M, a_inv: M, b_inv: M) {
    a * a_inv = M.1 and a_inv * a = M.1 and b * b_inv = M.1 and b_inv * b = M.1
        implies (b_inv * a_inv) * (a * b) = M.1
} by {
    if a * a_inv = M.1 and a_inv * a = M.1 and b * b_inv = M.1 and b_inv * b = M.1 {
        let c: M = b_inv * a_inv
        c * (a * b) = (b_inv * a_inv) * (a * b)
        (b_inv * a_inv) * (a * b) = b_inv * (a_inv * (a * b))
        a_inv * (a * b) = (a_inv * a) * b
        a_inv * a = M.1
        (a_inv * a) * b = M.1 * b
        M.1 * b = b
        a_inv * (a * b) = b
        b_inv * (a_inv * (a * b)) = b_inv * b
        b_inv * b = M.1
        c * (a * b) = M.1
    }
}

/// The product of two monoid units is a monoid unit.
theorem monoid_unit_mul_is_monoid_unit[M: Monoid](a: M, b: M) {
    is_monoid_unit(a) and is_monoid_unit(b) implies is_monoid_unit(a * b)
} by {
    if is_monoid_unit(a) and is_monoid_unit(b) {
        let a_inv: M satisfy {
            a * a_inv = M.1 and a_inv * a = M.1
        }
        let b_inv: M satisfy {
            b * b_inv = M.1 and b_inv * b = M.1
        }
        let c: M = b_inv * a_inv
        monoid_unit_mul_right_inverse(a, b, a_inv, b_inv)
        monoid_unit_mul_left_inverse(a, b, a_inv, b_inv)
        (a * b) * c = M.1
        c * (a * b) = M.1
        is_monoid_unit_of_inverse(a * b, c)
        is_monoid_unit(a * b)
    }
}

/// Multiplying a monoid unit by a unit power gives another unit power.
theorem monoid_unit_pow_suc_is_monoid_unit[M: Monoid](a: M, k: Nat) {
    is_monoid_unit(a) and is_monoid_unit(a.pow(k)) implies is_monoid_unit(a.pow(k.suc))
} by {
    if is_monoid_unit(a) and is_monoid_unit(a.pow(k)) {
        a.pow(k.suc) = a * a.pow(k)
        monoid_unit_mul_is_monoid_unit(a, a.pow(k))
        is_monoid_unit(a.pow(k.suc))
    }
}

/// Every natural power of a monoid unit is a monoid unit.
theorem monoid_unit_pow_is_monoid_unit[M: Monoid](a: M, n: Nat) {
    is_monoid_unit(a) implies is_monoid_unit(a.pow(n))
} by {
    define p(k: Nat) -> Bool {
        not(is_monoid_unit(a)) or is_monoid_unit(a.pow(k))
    }

    if is_monoid_unit(a) {
        a.pow(Nat.0) = M.1
        one_is_monoid_unit[M]
        is_monoid_unit(a.pow(Nat.0))
        p(Nat.0)
    } else {
        p(Nat.0)
    }

    forall(k: Nat) {
        if p(k) {
            if is_monoid_unit(a) {
                is_monoid_unit(a.pow(k))
                monoid_unit_pow_suc_is_monoid_unit(a, k)
                is_monoid_unit(a.pow(k.suc))
                p(k.suc)
            } else {
                p(k.suc)
            }
        }
    }
    p(Nat.0) and forall(k: Nat) {
        p(k) implies p(k.suc)
    }
    alt_induction(p)
    forall(k: Nat) {
        p(k)
    }

    if is_monoid_unit(a) {
        p(n)
        is_monoid_unit(a.pow(n))
    }
}

/// A two-sided inverse for a monoid element is unique.
theorem monoid_inverse_unique[M: Monoid](a: M, b: M, c: M) {
    a * b = M.1 and b * a = M.1 and a * c = M.1 and c * a = M.1 implies b = c
} by {
    if a * b = M.1 and b * a = M.1 and a * c = M.1 and c * a = M.1 {
        b = b * M.1
        b * M.1 = b * (a * c)
        b * (a * c) = (b * a) * c
        b * a = M.1
        (b * a) * c = M.1 * c
        M.1 * c = c
        b = c
    }
}

/// A right inverse for the value of a unit is the stored inverse.
theorem unit_right_inverse_unique[M: Monoid](u: Unit[M], b: M) {
    u.val * b = M.1 implies b = u.inv
} by {
    if u.val * b = M.1 {
        b = M.1 * b
        M.1 = u.inv * u.val
        M.1 * b = (u.inv * u.val) * b
        (u.inv * u.val) * b = u.inv * (u.val * b)
        u.val * b = M.1
        u.inv * (u.val * b) = u.inv * M.1
        u.inv * M.1 = u.inv
        b = u.inv
    }
}

/// A left inverse for the value of a unit is the stored inverse.
theorem unit_left_inverse_unique[M: Monoid](u: Unit[M], b: M) {
    b * u.val = M.1 implies b = u.inv
} by {
    if b * u.val = M.1 {
        b = b * M.1
        M.1 = u.val * u.inv
        b * M.1 = b * (u.val * u.inv)
        b * (u.val * u.inv) = (b * u.val) * u.inv
        b * u.val = M.1
        (b * u.val) * u.inv = M.1 * u.inv
        M.1 * u.inv = u.inv
        b = u.inv
    }
}

/// An inverse for a unit is unique.
theorem unit_inverse_unique[M: Monoid](u: Unit[M], b: M) {
    u.val * b = M.1 and b * u.val = M.1 implies b = u.inv
} by {
    if u.val * b = M.1 and b * u.val = M.1 {
        unit_right_inverse_unique(u, b)
        b = u.inv
    }
}

/// Equal unit values force equal stored inverses.
theorem unit_val_eq_inv_eq[M: Monoid](u: Unit[M], v: Unit[M]) {
    u.val = v.val implies u.inv = v.inv
} by {
    if u.val = v.val {
        v.val = u.val
        v.val * u.inv = u.val * u.inv
        u.val * u.inv = M.1
        v.val * u.inv = M.1
        unit_right_inverse_unique(v, u.inv)
        u.inv = v.inv
    }
}

/// Equal unit inverse components force equal values.
theorem unit_inv_eq_val_eq[M: Monoid](u: Unit[M], v: Unit[M]) {
    u.inv = v.inv implies u.val = v.val
} by {
    if u.inv = v.inv {
        v.inv = u.inv
        v.inv * u.val = u.inv * u.val
        u.inv * u.val = M.1
        v.inv * u.val = M.1
        u.val * v.inv = u.val * u.inv
        u.val * u.inv = M.1
        u.val * v.inv = M.1
        v.inv * v.val = M.1
        v.val * v.inv = M.1
        monoid_inverse_unique(v.inv, u.val, v.val)
        u.val = v.val
    }
}

/// Units are equal when their values are equal.
theorem unit_ext_val[M: Monoid](u: Unit[M], v: Unit[M]) {
    u.val = v.val implies u = v
} by {
    if u.val = v.val {
        unit_val_eq_inv_eq(u, v)
        u.inv = v.inv
        unit_ext(u, v)
    }
}

/// Units are equal when their inverse components are equal.
theorem unit_ext_inv[M: Monoid](u: Unit[M], v: Unit[M]) {
    u.inv = v.inv implies u = v
} by {
    if u.inv = v.inv {
        unit_inv_eq_val_eq(u, v)
        u.val = v.val
        unit_ext(u, v)
    }
}

/// Equality of units is equivalent to equality of their values.
theorem unit_eq_iff_val_eq[M: Monoid](u: Unit[M], v: Unit[M]) {
    (u = v) = (u.val = v.val)
} by {
    if u = v {
        u.val = v.val
    }
    if u.val = v.val {
        unit_ext_val(u, v)
        u = v
    }
}

/// Equality of units is equivalent to equality of their inverse components.
theorem unit_eq_iff_inv_eq[M: Monoid](u: Unit[M], v: Unit[M]) {
    (u = v) = (u.inv = v.inv)
} by {
    if u = v {
        u.inv = v.inv
    }
    if u.inv = v.inv {
        unit_ext_inv(u, v)
        u = v
    }
}

/// Equal units have equal values.
theorem unit_eq_val[M: Monoid](u: Unit[M], v: Unit[M]) {
    u = v implies u.val = v.val
}

/// Equal units have equal inverses.
theorem unit_eq_inv[M: Monoid](u: Unit[M], v: Unit[M]) {
    u = v implies u.inv = v.inv
}

/// The identity element as a unit.
let unit_one[M: Monoid]: Unit[M] satisfy {
    Unit.new(M.1, M.1) = Option.some(unit_one)
}

/// The product of two units.
let unit_mul[M: Monoid](u: Unit[M], v: Unit[M]) -> result: Unit[M] satisfy {
    Unit.new(u.val * v.val, v.inv * u.inv) = Option.some(result)
} by {
    (u.val * v.val) * (v.inv * u.inv) = u.val * (v.val * v.inv) * u.inv
    v.val * v.inv = M.1
    u.val * (v.val * v.inv) * u.inv = u.val * M.1 * u.inv
    u.val * M.1 = u.val
    u.val * M.1 * u.inv = u.val * u.inv
    u.val * u.inv = M.1
    (u.val * v.val) * (v.inv * u.inv) = M.1

    (v.inv * u.inv) * (u.val * v.val) = v.inv * (u.inv * u.val) * v.val
    u.inv * u.val = M.1
    v.inv * (u.inv * u.val) * v.val = v.inv * M.1 * v.val
    v.inv * M.1 = v.inv
    v.inv * M.1 * v.val = v.inv * v.val
    v.inv * v.val = M.1
    (v.inv * u.inv) * (u.val * v.val) = M.1
}

/// The inverse unit.
let unit_inverse[M: Monoid](u: Unit[M]) -> result: Unit[M] satisfy {
    Unit.new(u.inv, u.val) = Option.some(result)
} by {
    u.inv * u.val = M.1
    u.val * u.inv = M.1
}

/// The value of the identity unit is the monoid identity.
theorem unit_one_val[M: Monoid] {
    unit_one[M].val = M.1
}

/// The inverse of the identity unit is the monoid identity.
theorem unit_one_inv[M: Monoid] {
    unit_one[M].inv = M.1
}

/// The value of a product of units is the product of their values.
theorem unit_mul_val[M: Monoid](u: Unit[M], v: Unit[M]) {
    unit_mul(u, v).val = u.val * v.val
}

/// The inverse of a product of units is the reverse product of their inverses.
theorem unit_mul_inv[M: Monoid](u: Unit[M], v: Unit[M]) {
    unit_mul(u, v).inv = v.inv * u.inv
}

/// The product of the values of two units is a unit.
theorem unit_mul_val_is_monoid_unit[M: Monoid](u: Unit[M], v: Unit[M]) {
    is_monoid_unit(u.val * v.val)
} by {
    unit_val_is_monoid_unit(unit_mul(u, v))
    unit_mul(u, v).val = u.val * v.val
    is_monoid_unit(u.val * v.val)
}

/// The value of the inverse unit is the inverse of the original unit.
theorem unit_inverse_val[M: Monoid](u: Unit[M]) {
    unit_inverse(u).val = u.inv
}

/// The inverse of the inverse unit is the value of the original unit.
theorem unit_inverse_inv[M: Monoid](u: Unit[M]) {
    unit_inverse(u).inv = u.val
}

/// The inverse of a unit is a unit.
theorem unit_inv_is_monoid_unit[M: Monoid](u: Unit[M]) {
    is_monoid_unit(u.inv)
} by {
    unit_val_is_monoid_unit(unit_inverse(u))
    unit_inverse(u).val = u.inv
    is_monoid_unit(u.inv)
}

/// Multiplying a unit by the identity unit on the right changes nothing.
theorem unit_mul_one_right[M: Monoid](u: Unit[M]) {
    unit_mul(u, unit_one[M]) = u
} by {
    let lhs = unit_mul(u, unit_one[M])
    lhs.val = u.val * unit_one[M].val
    unit_one[M].val = M.1
    lhs.val = u.val * M.1
    lhs.val = u.val
    lhs.inv = unit_one[M].inv * u.inv
    unit_one[M].inv = M.1
    lhs.inv = M.1 * u.inv
    lhs.inv = u.inv
    unit_ext(lhs, u)
}

/// Multiplying a unit by the identity unit on the left changes nothing.
theorem unit_mul_one_left[M: Monoid](u: Unit[M]) {
    unit_mul(unit_one[M], u) = u
} by {
    let lhs = unit_mul(unit_one[M], u)
    lhs.val = unit_one[M].val * u.val
    unit_one[M].val = M.1
    lhs.val = M.1 * u.val
    lhs.val = u.val
    lhs.inv = u.inv * unit_one[M].inv
    unit_one[M].inv = M.1
    lhs.inv = u.inv * M.1
    lhs.inv = u.inv
    unit_ext(lhs, u)
}

/// A unit is the identity unit exactly when its value is the monoid identity.
theorem unit_eq_one_iff_val_eq_one[M: Monoid](u: Unit[M]) {
    (u = unit_one[M]) = (u.val = M.1)
} by {
    if u = unit_one[M] {
        u.val = unit_one[M].val
        unit_one[M].val = M.1
        u.val = M.1
    }
    if u.val = M.1 {
        unit_one[M].val = M.1
        u.val = unit_one[M].val
        unit_ext_val(u, unit_one[M])
        u = unit_one[M]
    }
}

/// A unit is the identity unit exactly when its inverse component is the monoid identity.
theorem unit_eq_one_iff_inv_eq_one[M: Monoid](u: Unit[M]) {
    (u = unit_one[M]) = (u.inv = M.1)
} by {
    if u = unit_one[M] {
        u.inv = unit_one[M].inv
        unit_one[M].inv = M.1
        u.inv = M.1
    }
    if u.inv = M.1 {
        unit_one[M].inv = M.1
        u.inv = unit_one[M].inv
        unit_ext_inv(u, unit_one[M])
        u = unit_one[M]
    }
}


/// Multiplying a unit by its inverse on the right gives the identity unit.
theorem unit_mul_inverse_right[M: Monoid](u: Unit[M]) {
    unit_mul(u, unit_inverse(u)) = unit_one[M]
} by {
    let lhs = unit_mul(u, unit_inverse(u))
    lhs.val = u.val * unit_inverse(u).val
    unit_inverse(u).val = u.inv
    lhs.val = u.val * u.inv
    lhs.val = M.1
    lhs.inv = unit_inverse(u).inv * u.inv
    unit_inverse(u).inv = u.val
    lhs.inv = u.val * u.inv
    lhs.inv = M.1
    unit_ext(lhs, unit_one[M])
}

/// Multiplying a unit by its inverse on the left gives the identity unit.
theorem unit_mul_inverse_left[M: Monoid](u: Unit[M]) {
    unit_mul(unit_inverse(u), u) = unit_one[M]
} by {
    let lhs = unit_mul(unit_inverse(u), u)
    lhs.val = unit_inverse(u).val * u.val
    unit_inverse(u).val = u.inv
    lhs.val = u.inv * u.val
    lhs.val = M.1
    lhs.inv = u.inv * unit_inverse(u).inv
    unit_inverse(u).inv = u.val
    lhs.inv = u.inv * u.val
    lhs.inv = M.1
    unit_ext(lhs, unit_one[M])
}

/// The value of a product is the identity exactly when the second factor is the inverse of the first.
theorem unit_mul_eq_one_iff_eq_inverse[M: Monoid](u: Unit[M], v: Unit[M]) {
    (unit_mul(u, v) = unit_one[M]) = (v = unit_inverse(u))
} by {
    if unit_mul(u, v) = unit_one[M] {
        unit_eq_val(unit_mul(u, v), unit_one[M])
        unit_mul(u, v).val = unit_one[M].val
        unit_mul_val(u, v)
        unit_one[M].val = M.1
        u.val * v.val = M.1
        unit_right_inverse_unique(u, v.val)
        v.val = u.inv
        unit_inverse(u).val = u.inv
        v.val = unit_inverse(u).val
        unit_ext_val(v, unit_inverse(u))
        v = unit_inverse(u)
    }
    if v = unit_inverse(u) {
        unit_mul_inverse_right(u)
        unit_mul(u, v) = unit_one[M]
    }
}

/// The value of a reversed product is the identity exactly when the first factor is the inverse of the second.
theorem unit_mul_eq_one_iff_eq_inverse_left[M: Monoid](u: Unit[M], v: Unit[M]) {
    (unit_mul(u, v) = unit_one[M]) = (u = unit_inverse(v))
} by {
    if unit_mul(u, v) = unit_one[M] {
        unit_eq_val(unit_mul(u, v), unit_one[M])
        unit_mul(u, v).val = unit_one[M].val
        unit_mul_val(u, v)
        unit_one[M].val = M.1
        u.val * v.val = M.1
        unit_left_inverse_unique(v, u.val)
        u.val = v.inv
        unit_inverse(v).val = v.inv
        u.val = unit_inverse(v).val
        unit_ext_val(u, unit_inverse(v))
        u = unit_inverse(v)
    }
    if u = unit_inverse(v) {
        unit_mul_inverse_left(v)
        unit_mul(unit_inverse(v), v) = unit_one[M]
        unit_mul(u, v) = unit_one[M]
    }
}

/// Unit multiplication is associative.
theorem unit_mul_assoc[M: Monoid](u: Unit[M], v: Unit[M], w: Unit[M]) {
    unit_mul(u, unit_mul(v, w)) = unit_mul(unit_mul(u, v), w)
} by {
    let lhs = unit_mul(u, unit_mul(v, w))
    let rhs = unit_mul(unit_mul(u, v), w)
    lhs.val = u.val * unit_mul(v, w).val
    unit_mul(v, w).val = v.val * w.val
    lhs.val = u.val * (v.val * w.val)
    rhs.val = unit_mul(u, v).val * w.val
    unit_mul(u, v).val = u.val * v.val
    rhs.val = (u.val * v.val) * w.val
    lhs.val = rhs.val
    lhs.inv = unit_mul(v, w).inv * u.inv
    unit_mul(v, w).inv = w.inv * v.inv
    lhs.inv = (w.inv * v.inv) * u.inv
    rhs.inv = w.inv * unit_mul(u, v).inv
    unit_mul(u, v).inv = v.inv * u.inv
    rhs.inv = w.inv * (v.inv * u.inv)
    lhs.inv = rhs.inv
    unit_ext(lhs, rhs)
}

/// Inverting a unit twice gives the original unit.
theorem unit_inverse_inverse[M: Monoid](u: Unit[M]) {
    unit_inverse(unit_inverse(u)) = u
} by {
    let lhs = unit_inverse(unit_inverse(u))
    lhs.val = unit_inverse(u).inv
    unit_inverse(u).inv = u.val
    lhs.val = u.val
    lhs.inv = unit_inverse(u).val
    unit_inverse(u).val = u.inv
    lhs.inv = u.inv
    unit_ext(lhs, u)
}

/// Inversion reverses multiplication of units.
theorem unit_inverse_mul[M: Monoid](u: Unit[M], v: Unit[M]) {
    unit_inverse(unit_mul(u, v)) = unit_mul(unit_inverse(v), unit_inverse(u))
} by {
    let lhs = unit_inverse(unit_mul(u, v))
    let rhs = unit_mul(unit_inverse(v), unit_inverse(u))
    lhs.val = unit_mul(u, v).inv
    unit_mul(u, v).inv = v.inv * u.inv
    lhs.val = v.inv * u.inv
    rhs.val = unit_inverse(v).val * unit_inverse(u).val
    unit_inverse(v).val = v.inv
    unit_inverse(u).val = u.inv
    rhs.val = v.inv * u.inv
    lhs.val = rhs.val
    lhs.inv = unit_mul(u, v).val
    unit_mul(u, v).val = u.val * v.val
    lhs.inv = u.val * v.val
    rhs.inv = unit_inverse(u).inv * unit_inverse(v).inv
    unit_inverse(u).inv = u.val
    unit_inverse(v).inv = v.val
    rhs.inv = u.val * v.val
    lhs.inv = rhs.inv
    unit_ext(lhs, rhs)
}

/// A monoid homomorphism carries monoid units to monoid units.
theorem monoid_hom_maps_monoid_unit[M: Monoid, N: Monoid](f: MonoidHom[M, N], a: M) {
    is_monoid_unit(a) implies is_monoid_unit(f.hom(a))
} by {
    if is_monoid_unit(a) {
        let b: M satisfy {
            a * b = M.1 and b * a = M.1
        }
        monoid_hom_mul(f, a, b)
        f.hom(a * b) = f.hom(a) * f.hom(b)
        a * b = M.1
        f.hom(a * b) = f.hom(M.1)
        monoid_hom_one(f)
        f.hom(M.1) = N.1
        f.hom(a) * f.hom(b) = N.1

        monoid_hom_mul(f, b, a)
        f.hom(b * a) = f.hom(b) * f.hom(a)
        b * a = M.1
        f.hom(b * a) = f.hom(M.1)
        f.hom(b) * f.hom(a) = N.1
        exists(c: N) {
            c = f.hom(b) and f.hom(a) * c = N.1 and c * f.hom(a) = N.1
        }
    }
}

/// The image of a unit under a monoid homomorphism.
let monoid_hom_unit[M: Monoid, N: Monoid](f: MonoidHom[M, N], u: Unit[M]) -> result: Unit[N] satisfy {
    Unit.new(f.hom(u.val), f.hom(u.inv)) = Option.some(result)
} by {
    monoid_hom_mul(f, u.val, u.inv)
    f.hom(u.val * u.inv) = f.hom(u.val) * f.hom(u.inv)
    u.val * u.inv = M.1
    f.hom(u.val * u.inv) = f.hom(M.1)
    monoid_hom_one(f)
    f.hom(M.1) = N.1
    f.hom(u.val) * f.hom(u.inv) = N.1

    monoid_hom_mul(f, u.inv, u.val)
    f.hom(u.inv * u.val) = f.hom(u.inv) * f.hom(u.val)
    u.inv * u.val = M.1
    f.hom(u.inv * u.val) = f.hom(M.1)
    f.hom(u.inv) * f.hom(u.val) = N.1
}

/// The value of the image unit is the image of the value.
theorem monoid_hom_unit_val[M: Monoid, N: Monoid](f: MonoidHom[M, N], u: Unit[M]) {
    monoid_hom_unit(f, u).val = f.hom(u.val)
}

/// The inverse of the image unit is the image of the inverse.
theorem monoid_hom_unit_inv[M: Monoid, N: Monoid](f: MonoidHom[M, N], u: Unit[M]) {
    monoid_hom_unit(f, u).inv = f.hom(u.inv)
}

/// A monoid homomorphism carries the identity unit to the identity unit.
theorem monoid_hom_unit_one[M: Monoid, N: Monoid](f: MonoidHom[M, N]) {
    monoid_hom_unit(f, unit_one[M]) = unit_one[N]
} by {
    let lhs = monoid_hom_unit(f, unit_one[M])
    lhs.val = f.hom(unit_one[M].val)
    unit_one[M].val = M.1
    monoid_hom_one(f)
    lhs.val = N.1
    lhs.inv = f.hom(unit_one[M].inv)
    unit_one[M].inv = M.1
    lhs.inv = f.hom(M.1)
    lhs.inv = N.1
    unit_ext(lhs, unit_one[N])
}

/// The identity monoid homomorphism acts trivially on units.
theorem monoid_hom_unit_identity[M: Monoid](u: Unit[M]) {
    monoid_hom_unit(identity_monoid_hom[M], u) = u
} by {
    let lhs = monoid_hom_unit(identity_monoid_hom[M], u)
    lhs.val = identity_monoid_hom[M].hom(u.val)
    identity_monoid_hom_hom[M]
    identity_monoid_hom[M].hom = identity_fn[M]
    lhs.val = identity_fn[M](u.val)
    lhs.val = u.val
    lhs.inv = identity_monoid_hom[M].hom(u.inv)
    lhs.inv = identity_fn[M](u.inv)
    lhs.inv = u.inv
    unit_ext(lhs, u)
}

/// Transporting a unit along a composite monoid homomorphism equals successive
/// transport along the two homomorphisms.
theorem monoid_hom_unit_compose[M: Monoid, N: Monoid, P: Monoid](
    f: MonoidHom[N, P],
    g: MonoidHom[M, N],
    u: Unit[M]
) {
    monoid_hom_unit(compose_monoid_hom(f, g), u) = monoid_hom_unit(f, monoid_hom_unit(g, u))
} by {
    let lhs = monoid_hom_unit(compose_monoid_hom(f, g), u)
    let rhs = monoid_hom_unit(f, monoid_hom_unit(g, u))
    lhs.val = compose_monoid_hom(f, g).hom(u.val)
    compose_monoid_hom_hom(f, g)
    compose_monoid_hom(f, g).hom = compose(f.hom, g.hom)
    lhs.val = compose(f.hom, g.hom)(u.val)
    lhs.val = f.hom(g.hom(u.val))
    rhs.val = f.hom(monoid_hom_unit(g, u).val)
    monoid_hom_unit(g, u).val = g.hom(u.val)
    rhs.val = f.hom(g.hom(u.val))
    lhs.val = rhs.val

    lhs.inv = compose_monoid_hom(f, g).hom(u.inv)
    lhs.inv = compose(f.hom, g.hom)(u.inv)
    lhs.inv = f.hom(g.hom(u.inv))
    rhs.inv = f.hom(monoid_hom_unit(g, u).inv)
    monoid_hom_unit(g, u).inv = g.hom(u.inv)
    rhs.inv = f.hom(g.hom(u.inv))
    lhs.inv = rhs.inv
    unit_ext(lhs, rhs)
}

/// Transporting the inverse unit gives the inverse of the transported unit.
theorem monoid_hom_unit_inverse[M: Monoid, N: Monoid](f: MonoidHom[M, N], u: Unit[M]) {
    monoid_hom_unit(f, unit_inverse(u)) = unit_inverse(monoid_hom_unit(f, u))
} by {
    let lhs = monoid_hom_unit(f, unit_inverse(u))
    let rhs = unit_inverse(monoid_hom_unit(f, u))
    lhs.val = f.hom(unit_inverse(u).val)
    unit_inverse(u).val = u.inv
    lhs.val = f.hom(u.inv)
    rhs.val = monoid_hom_unit(f, u).inv
    monoid_hom_unit(f, u).inv = f.hom(u.inv)
    lhs.val = rhs.val
    lhs.inv = f.hom(unit_inverse(u).inv)
    unit_inverse(u).inv = u.val
    lhs.inv = f.hom(u.val)
    rhs.inv = monoid_hom_unit(f, u).val
    monoid_hom_unit(f, u).val = f.hom(u.val)
    lhs.inv = rhs.inv
    unit_ext(lhs, rhs)
}

/// Transporting a product of units gives the product of the transported units.
theorem monoid_hom_unit_mul[M: Monoid, N: Monoid](f: MonoidHom[M, N], u: Unit[M], v: Unit[M]) {
    monoid_hom_unit(f, unit_mul(u, v)) = unit_mul(monoid_hom_unit(f, u), monoid_hom_unit(f, v))
} by {
    let lhs = monoid_hom_unit(f, unit_mul(u, v))
    let rhs = unit_mul(monoid_hom_unit(f, u), monoid_hom_unit(f, v))
    lhs.val = f.hom(unit_mul(u, v).val)
    unit_mul(u, v).val = u.val * v.val
    lhs.val = f.hom(u.val * v.val)
    monoid_hom_mul(f, u.val, v.val)
    lhs.val = f.hom(u.val) * f.hom(v.val)
    rhs.val = monoid_hom_unit(f, u).val * monoid_hom_unit(f, v).val
    monoid_hom_unit(f, u).val = f.hom(u.val)
    monoid_hom_unit(f, v).val = f.hom(v.val)
    rhs.val = f.hom(u.val) * f.hom(v.val)
    lhs.val = rhs.val

    lhs.inv = f.hom(unit_mul(u, v).inv)
    unit_mul(u, v).inv = v.inv * u.inv
    lhs.inv = f.hom(v.inv * u.inv)
    monoid_hom_mul(f, v.inv, u.inv)
    lhs.inv = f.hom(v.inv) * f.hom(u.inv)
    rhs.inv = monoid_hom_unit(f, v).inv * monoid_hom_unit(f, u).inv
    monoid_hom_unit(f, v).inv = f.hom(v.inv)
    monoid_hom_unit(f, u).inv = f.hom(u.inv)
    rhs.inv = f.hom(v.inv) * f.hom(u.inv)
    lhs.inv = rhs.inv
    unit_ext(lhs, rhs)
}

/// Multiplication by the value of a unit on the left has trivial zero kernel.
theorem unit_val_mul_eq_zero_left[R: Ring](u: Unit[R], a: R) {
    u.val * a = R.0 implies a = R.0
} by {
    if u.val * a = R.0 {
        a = R.1 * a
        R.1 = u.inv * u.val
        R.1 * a = (u.inv * u.val) * a
        (u.inv * u.val) * a = u.inv * (u.val * a)
        u.inv * (u.val * a) = u.inv * R.0
        u.inv * R.0 = R.0
        a = R.0
    }
}

/// Multiplication by the value of a unit on the right has trivial zero kernel.
theorem unit_val_mul_eq_zero_right[R: Ring](u: Unit[R], a: R) {
    a * u.val = R.0 implies a = R.0
} by {
    if a * u.val = R.0 {
        a = a * R.1
        R.1 = u.val * u.inv
        a * R.1 = a * (u.val * u.inv)
        a * (u.val * u.inv) = (a * u.val) * u.inv
        (a * u.val) * u.inv = R.0 * u.inv
        R.0 * u.inv = R.0
        a = R.0
    }
}

/// A unit value cannot annihilate a nonzero element on the left.
theorem unit_val_mul_nonzero_left[R: Ring](u: Unit[R], a: R) {
    a != R.0 implies u.val * a != R.0
} by {
    if a != R.0 {
        if u.val * a = R.0 {
            unit_val_mul_eq_zero_left(u, a)
            a = R.0
        }
    }
}

/// A unit value cannot annihilate a nonzero element on the right.
theorem unit_val_mul_nonzero_right[R: Ring](u: Unit[R], a: R) {
    a != R.0 implies a * u.val != R.0
} by {
    if a != R.0 {
        if a * u.val = R.0 {
            unit_val_mul_eq_zero_right(u, a)
            a = R.0
        }
    }
}

/// A product by a unit value on the left is zero exactly when the other factor is zero.
theorem unit_val_mul_zero_iff_left[R: Ring](u: Unit[R], a: R) {
    (u.val * a = R.0) = (a = R.0)
} by {
    if u.val * a = R.0 {
        unit_val_mul_eq_zero_left(u, a)
        a = R.0
    } else {
        if a = R.0 {
            u.val * a = u.val * R.0
            u.val * R.0 = R.0
        }
    }
}

/// A product by a unit value on the right is zero exactly when the other factor is zero.
theorem unit_val_mul_zero_iff_right[R: Ring](u: Unit[R], a: R) {
    (a * u.val = R.0) = (a = R.0)
} by {
    if a * u.val = R.0 {
        unit_val_mul_eq_zero_right(u, a)
        a = R.0
    } else {
        if a = R.0 {
            a * u.val = R.0 * u.val
            R.0 * u.val = R.0
        }
    }
}

/// Multiplication by the value of a unit on the left is cancellative.
theorem unit_val_mul_cancel_left[R: Ring](u: Unit[R], a: R, b: R) {
    u.val * a = u.val * b implies a = b
} by {
    if u.val * a = u.val * b {
        a = R.1 * a
        R.1 = u.inv * u.val
        R.1 * a = (u.inv * u.val) * a
        (u.inv * u.val) * a = u.inv * (u.val * a)
        b = R.1 * b
        R.1 = u.inv * u.val
        R.1 * b = (u.inv * u.val) * b
        (u.inv * u.val) * b = u.inv * (u.val * b)
        u.inv * (u.val * a) = u.inv * (u.val * b)
        a = b
    }
}

/// Multiplication by the value of a unit on the right is cancellative.
theorem unit_val_mul_cancel_right[R: Ring](u: Unit[R], a: R, b: R) {
    a * u.val = b * u.val implies a = b
} by {
    if a * u.val = b * u.val {
        a = a * R.1
        R.1 = u.val * u.inv
        a * R.1 = a * (u.val * u.inv)
        a * (u.val * u.inv) = (a * u.val) * u.inv
        b = b * R.1
        R.1 = u.val * u.inv
        b * R.1 = b * (u.val * u.inv)
        b * (u.val * u.inv) = (b * u.val) * u.inv
        (a * u.val) * u.inv = (b * u.val) * u.inv
        a = b
    }
}

/// Multiplication by a monoid unit on the left has trivial zero kernel.
theorem monoid_unit_mul_eq_zero_left[R: Ring](u: R, a: R) {
    is_monoid_unit(u) and u * a = R.0 implies a = R.0
} by {
    if is_monoid_unit(u) and u * a = R.0 {
        let v: R satisfy {
            u * v = R.1 and v * u = R.1
        }
        a = R.1 * a
        R.1 = v * u
        R.1 * a = (v * u) * a
        (v * u) * a = v * (u * a)
        v * (u * a) = v * R.0
        v * R.0 = R.0
        a = R.0
    }
}

/// Multiplication by a monoid unit on the right has trivial zero kernel.
theorem monoid_unit_mul_eq_zero_right[R: Ring](u: R, a: R) {
    is_monoid_unit(u) and a * u = R.0 implies a = R.0
} by {
    if is_monoid_unit(u) and a * u = R.0 {
        let v: R satisfy {
            u * v = R.1 and v * u = R.1
        }
        a = a * R.1
        R.1 = u * v
        a * R.1 = a * (u * v)
        a * (u * v) = (a * u) * v
        (a * u) * v = R.0 * v
        R.0 * v = R.0
        a = R.0
    }
}

/// A monoid unit cannot annihilate a nonzero element on the left.
theorem monoid_unit_mul_nonzero_left[R: Ring](u: R, a: R) {
    is_monoid_unit(u) and a != R.0 implies u * a != R.0
} by {
    if is_monoid_unit(u) and a != R.0 {
        if u * a = R.0 {
            monoid_unit_mul_eq_zero_left(u, a)
            a = R.0
        }
    }
}

/// A monoid unit cannot annihilate a nonzero element on the right.
theorem monoid_unit_mul_nonzero_right[R: Ring](u: R, a: R) {
    is_monoid_unit(u) and a != R.0 implies a * u != R.0
} by {
    if is_monoid_unit(u) and a != R.0 {
        if a * u = R.0 {
            monoid_unit_mul_eq_zero_right(u, a)
            a = R.0
        }
    }
}

/// Multiplication by a monoid unit on the left is cancellative.
theorem monoid_unit_mul_cancel_left[R: Ring](u: R, a: R, b: R) {
    is_monoid_unit(u) and u * a = u * b implies a = b
} by {
    if is_monoid_unit(u) and u * a = u * b {
        let v: R satisfy {
            u * v = R.1 and v * u = R.1
        }
        a = R.1 * a
        R.1 = v * u
        R.1 * a = (v * u) * a
        (v * u) * a = v * (u * a)
        b = R.1 * b
        R.1 = v * u
        R.1 * b = (v * u) * b
        (v * u) * b = v * (u * b)
        v * (u * a) = v * (u * b)
        a = b
    }
}

/// Multiplication by a monoid unit on the right is cancellative.
theorem monoid_unit_mul_cancel_right[R: Ring](u: R, a: R, b: R) {
    is_monoid_unit(u) and a * u = b * u implies a = b
} by {
    if is_monoid_unit(u) and a * u = b * u {
        let v: R satisfy {
            u * v = R.1 and v * u = R.1
        }
        a = a * R.1
        R.1 = u * v
        a * R.1 = a * (u * v)
        a * (u * v) = (a * u) * v
        b = b * R.1
        R.1 = u * v
        b * R.1 = b * (u * v)
        b * (u * v) = (b * u) * v
        (a * u) * v = (b * u) * v
        a = b
    }
}

/// A product by a monoid unit on the left is zero exactly when the other factor is zero.
theorem monoid_unit_mul_zero_iff_left[R: Ring](u: R, a: R) {
    is_monoid_unit(u) implies ((u * a = R.0) = (a = R.0))
} by {
    if is_monoid_unit(u) {
        if u * a = R.0 {
            is_monoid_unit(u) and u * a = R.0
            monoid_unit_mul_eq_zero_left(u, a)
            a = R.0
        }
        if a = R.0 {
            u * a = u * R.0
            u * R.0 = R.0
            u * a = R.0
        }
    }
}

/// A product by a monoid unit on the right is zero exactly when the other factor is zero.
theorem monoid_unit_mul_zero_iff_right[R: Ring](u: R, a: R) {
    is_monoid_unit(u) implies ((a * u = R.0) = (a = R.0))
} by {
    if is_monoid_unit(u) {
        if a * u = R.0 {
            is_monoid_unit(u) and a * u = R.0
            monoid_unit_mul_eq_zero_right(u, a)
            a = R.0
        }
        if a = R.0 {
            a * u = R.0 * u
            R.0 * u = R.0
            a * u = R.0
        }
    }
}

/// Left multiplication by a unit value preserves and reflects equality.
theorem unit_val_mul_eq_iff_left[R: Ring](u: Unit[R], a: R, b: R) {
    (u.val * a = u.val * b) = (a = b)
} by {
    if u.val * a = u.val * b {
        unit_val_mul_cancel_left(u, a, b)
        a = b
    }
    if a = b {
        u.val * a = u.val * b
    }
}

/// Right multiplication by a unit value preserves and reflects equality.
theorem unit_val_mul_eq_iff_right[R: Ring](u: Unit[R], a: R, b: R) {
    (a * u.val = b * u.val) = (a = b)
} by {
    if a * u.val = b * u.val {
        unit_val_mul_cancel_right(u, a, b)
        a = b
    }
    if a = b {
        a * u.val = b * u.val
    }
}

/// Left multiplication by a monoid unit preserves and reflects equality.
theorem monoid_unit_mul_eq_iff_left[R: Ring](u: R, a: R, b: R) {
    is_monoid_unit(u) implies ((u * a = u * b) = (a = b))
} by {
    if is_monoid_unit(u) {
        if u * a = u * b {
            monoid_unit_mul_cancel_left(u, a, b)
            a = b
        }
        if a = b {
            u * a = u * b
        }
    }
}

/// Right multiplication by a monoid unit preserves and reflects equality.
theorem monoid_unit_mul_eq_iff_right[R: Ring](u: R, a: R, b: R) {
    is_monoid_unit(u) implies ((a * u = b * u) = (a = b))
} by {
    if is_monoid_unit(u) {
        if a * u = b * u {
            monoid_unit_mul_cancel_right(u, a, b)
            a = b
        }
        if a = b {
            a * u = b * u
        }
    }
}

/// Left multiplication by a unit value reflects inequality.
theorem unit_val_mul_ne_left[R: Ring](u: Unit[R], a: R, b: R) {
    a != b implies u.val * a != u.val * b
} by {
    if a != b {
        if u.val * a = u.val * b {
            unit_val_mul_cancel_left(u, a, b)
            a = b
        }
    }
}

/// Right multiplication by a unit value reflects inequality.
theorem unit_val_mul_ne_right[R: Ring](u: Unit[R], a: R, b: R) {
    a != b implies a * u.val != b * u.val
} by {
    if a != b {
        if a * u.val = b * u.val {
            unit_val_mul_cancel_right(u, a, b)
            a = b
        }
    }
}

/// The value of a unit in a field is nonzero.
theorem field_unit_val_nonzero[F: Field](u: Unit[F]) {
    u.val != F.0
} by {
    if u.val = F.0 {
        F.1 = u.inv * u.val
        u.inv * u.val = u.inv * F.0
        u.inv * F.0 = F.0
        F.1 = F.0
    }
}

/// The inverse value of a unit in a field is nonzero.
theorem field_unit_inv_nonzero[F: Field](u: Unit[F]) {
    u.inv != F.0
} by {
    if u.inv = F.0 {
        F.1 = u.val * u.inv
        u.val * u.inv = u.val * F.0
        u.val * F.0 = F.0
        F.1 = F.0
    }
}

/// The inverse component of a field unit is the field inverse of its value.
theorem field_unit_inv_eq_inverse[F: Field](u: Unit[F]) {
    u.inv = u.val.inverse
} by {
    field_unit_val_nonzero(u)
    u.val != F.0
    u.val * u.inv = F.1
    unique_inverse(u.val, u.inv)
    u.inv = u.val.inverse
}

/// The value component of a field unit is the field inverse of its inverse component.
theorem field_unit_val_eq_inverse_inv[F: Field](u: Unit[F]) {
    u.val = u.inv.inverse
} by {
    field_unit_inv_nonzero(u)
    u.inv != F.0
    u.inv * u.val = F.1
    unique_inverse(u.inv, u.val)
    u.val = u.inv.inverse
}

/// The field inverse of the value of a unit is nonzero.
theorem field_unit_val_inverse_nonzero[F: Field](u: Unit[F]) {
    u.val.inverse != F.0
} by {
    field_unit_inv_nonzero(u)
    field_unit_inv_eq_inverse(u)
    u.inv = u.val.inverse
    u.val.inverse != F.0
}

/// A field unit value multiplied by a nonzero element on the left is nonzero.
theorem field_unit_val_mul_nonzero_left[F: Field](u: Unit[F], a: F) {
    a != F.0 implies u.val * a != F.0
} by {
    unit_val_mul_nonzero_left(u, a)
}

/// A field unit value multiplied by a nonzero element on the right is nonzero.
theorem field_unit_val_mul_nonzero_right[F: Field](u: Unit[F], a: F) {
    a != F.0 implies a * u.val != F.0
} by {
    unit_val_mul_nonzero_right(u, a)
}

/// Every nonzero element of a field is a monoid unit.
theorem field_nonzero_is_monoid_unit[F: Field](a: F) {
    a != F.0 implies is_monoid_unit(a)
} by {
    if a != F.0 {
        a * a.inverse = F.1
        a.inverse * a = F.1
        exists(b: F) {
            b = a.inverse and a * b = F.1 and b * a = F.1
        }
    }
}

/// The inverse of a nonzero field element is a monoid unit.
theorem field_inverse_is_monoid_unit[F: Field](a: F) {
    a != F.0 implies is_monoid_unit(a.inverse)
} by {
    if a != F.0 {
        a.inverse != F.0
        field_nonzero_is_monoid_unit(a.inverse)
        is_monoid_unit(a.inverse)
    }
}

/// A field element is a monoid unit exactly when it is nonzero.
theorem field_is_monoid_unit_iff_nonzero[F: Field](a: F) {
    is_monoid_unit(a) = (a != F.0)
} by {
    if is_monoid_unit(a) {
        let b: F satisfy {
            a * b = F.1 and b * a = F.1
        }
        if a = F.0 {
            F.1 = a * b
            a * b = F.0 * b
            F.0 * b = F.0
            F.1 = F.0
        }
    } else {
        if a != F.0 {
            field_nonzero_is_monoid_unit(a)
            is_monoid_unit(a)
        }
    }
}

/// Zero is not a monoid unit in a field.
theorem field_zero_not_monoid_unit[F: Field] {
    not(is_monoid_unit(F.0))
} by {
    if is_monoid_unit(F.0) {
        let b: F satisfy {
            F.0 * b = F.1 and b * F.0 = F.1
        }
        F.1 = F.0 * b
        F.0 * b = F.0
        F.1 = F.0
    }
}

/// One is a monoid unit in a field.
theorem field_one_is_monoid_unit[F: Field] {
    is_monoid_unit(F.1)
} by {
    one_is_monoid_unit[F]
}

/// A product equal to one in a field has nonzero left factor.
theorem field_mul_eq_one_left_nonzero[F: Field](a: F, b: F) {
    a * b = F.1 implies a != F.0
} by {
    if a * b = F.1 {
        if a = F.0 {
            F.1 = a * b
            a * b = F.0 * b
            F.0 * b = F.0
            F.1 = F.0
        }
    }
}

/// A product equal to one in a field has nonzero right factor.
theorem field_mul_eq_one_right_nonzero[F: Field](a: F, b: F) {
    a * b = F.1 implies b != F.0
} by {
    if a * b = F.1 {
        if b = F.0 {
            F.1 = a * b
            a * b = a * F.0
            a * F.0 = F.0
            F.1 = F.0
        }
    }
}

/// A nonzero field element has a unit with the field inverse as inverse.
theorem field_unit_exists_of_nonzero[F: Field](a: F) {
    a != F.0 implies exists(u: Unit[F]) {
        u.val = a and u.inv = a.inverse
    }
} by {
    if a != F.0 {
        a * a.inverse = F.1
        a.inverse * a = F.1
        let u: Unit[F] satisfy {
            Unit.new(a, a.inverse) = Option.some(u)
        }
        u.val = a
        u.inv = a.inverse
        exists(v: Unit[F]) {
            v = u and v.val = a and v.inv = a.inverse
        }
    }
}
