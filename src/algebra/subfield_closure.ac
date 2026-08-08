/// Generated subfields and least-subfield closure.

from algebra.field.field import Field
from data.basic.set import Set, subset_contains, subset_contains_eq
from algebra.subfield import Subfield, subfield_constraint, subfield_zero_constraint,
    subfield_one_constraint, subfield_add_constraint, subfield_neg_constraint,
    subfield_mul_constraint, subfield_inverse_constraint, subfield_additive_constraint,
    subfield_multiplicative_constraint, subfield_le, subfield_le_intro,
    subfield_le_antisymm, subfield_contains_zero, subfield_contains_one,
    subfield_contains_add, subfield_contains_neg, subfield_contains_mul,
    subfield_contains_inverse

/// The underlying set of a subfield.
define subfield_carrier[F: Field](s: Subfield[F]) -> Set[F] {
    Set[F].new(s.contains)
}

/// Membership in the carrier set of a subfield is subfield membership.
theorem subfield_carrier_contains_eq[F: Field](s: Subfield[F], x: F) {
    subfield_carrier(s).contains(x) = s.contains(x)
}

/// True if every member of a set of field elements lies in a subfield.
define set_subset_subfield[F: Field](a: Set[F], s: Subfield[F]) -> Bool {
    a.subset(subfield_carrier(s))
}

/// Introduction rule for containment of a set in a subfield.
theorem set_subset_subfield_intro[F: Field](a: Set[F], s: Subfield[F]) {
    (forall(x: F) { a.contains(x) implies s.contains(x) }) implies set_subset_subfield(a, s)
} by {
    if forall(x: F) { a.contains(x) implies s.contains(x) } {
        forall(x: F) {
            if a.contains(x) {
                s.contains(x)
                subfield_carrier_contains_eq(s, x)
                subfield_carrier(s).contains(x)
            }
        }
        subset_contains_eq(a, subfield_carrier(s))
        a.subset(subfield_carrier(s))
        set_subset_subfield(a, s)
    }
}

/// A set contained in a subfield carries membership into that subfield.
theorem set_subset_subfield_contains[F: Field](a: Set[F], s: Subfield[F], x: F) {
    set_subset_subfield(a, s) and a.contains(x) implies s.contains(x)
} by {
    if set_subset_subfield(a, s) and a.contains(x) {
        a.subset(subfield_carrier(s))
        subset_contains(a, subfield_carrier(s), x)
        subfield_carrier(s).contains(x)
        subfield_carrier_contains_eq(s, x)
        s.contains(x)
    }
}

/// A subfield contains its own carrier set.
theorem set_subset_subfield_carrier_self[F: Field](s: Subfield[F]) {
    set_subset_subfield(subfield_carrier(s), s)
} by {
    forall(x: F) {
        if subfield_carrier(s).contains(x) {
            subfield_carrier_contains_eq(s, x)
            s.contains(x)
        }
    }
    set_subset_subfield_intro(subfield_carrier(s), s)
}

/// The least-subfield membership predicate generated by a set of field elements.
define subfield_closure_contains[F: Field](a: Set[F], x: F) -> Bool {
    forall(s: Subfield[F]) {
        set_subset_subfield(a, s) implies s.contains(x)
    }
}

/// Introduction rule for membership in the generated subfield predicate.
theorem subfield_closure_contains_intro[F: Field](a: Set[F], x: F) {
    (forall(s: Subfield[F]) { set_subset_subfield(a, s) implies s.contains(x) })
    implies subfield_closure_contains(a, x)
} by {
    if forall(s: Subfield[F]) { set_subset_subfield(a, s) implies s.contains(x) } {
        subfield_closure_contains(a, x) = forall(s: Subfield[F]) {
            set_subset_subfield(a, s) implies s.contains(x)
        }
    }
}

/// Membership in the generated predicate lies in every subfield containing the generators.
theorem subfield_closure_contains_in_subfield[F: Field](a: Set[F], s: Subfield[F], x: F) {
    subfield_closure_contains(a, x) and set_subset_subfield(a, s) implies s.contains(x)
} by {
    if subfield_closure_contains(a, x) and set_subset_subfield(a, s) {
        subfield_closure_contains(a, x) = forall(t: Subfield[F]) {
            set_subset_subfield(a, t) implies t.contains(x)
        }
        set_subset_subfield(a, s) implies s.contains(x)
        s.contains(x)
    }
}

/// Zero lies in the generated subfield predicate.
theorem subfield_closure_contains_zero[F: Field](a: Set[F]) {
    subfield_closure_contains(a, F.0)
} by {
    forall(s: Subfield[F]) {
        if set_subset_subfield(a, s) {
            subfield_contains_zero(s)
            s.contains(F.0)
        }
    }
    subfield_closure_contains_intro(a, F.0)
}

/// One lies in the generated subfield predicate.
theorem subfield_closure_contains_one[F: Field](a: Set[F]) {
    subfield_closure_contains(a, F.1)
} by {
    forall(s: Subfield[F]) {
        if set_subset_subfield(a, s) {
            subfield_contains_one(s)
            s.contains(F.1)
        }
    }
    subfield_closure_contains_intro(a, F.1)
}

/// The generated subfield predicate is closed under addition.
theorem subfield_closure_contains_add[F: Field](a: Set[F], x: F, y: F) {
    subfield_closure_contains(a, x) and subfield_closure_contains(a, y)
    implies subfield_closure_contains(a, x + y)
} by {
    if subfield_closure_contains(a, x) and subfield_closure_contains(a, y) {
        forall(s: Subfield[F]) {
            if set_subset_subfield(a, s) {
                (subfield_closure_contains(a, x) and set_subset_subfield(a, s))
                subfield_closure_contains_in_subfield(a, s, x)
                s.contains(x)
                (subfield_closure_contains(a, y) and set_subset_subfield(a, s))
                subfield_closure_contains_in_subfield(a, s, y)
                s.contains(y)
                subfield_contains_add(s, x, y)
                s.contains(x + y)
            }
        }
        forall(s: Subfield[F]) {
            set_subset_subfield(a, s) implies s.contains(x + y)
        }
        subfield_closure_contains(a, x + y) = forall(s: Subfield[F]) {
            set_subset_subfield(a, s) implies s.contains(x + y)
        }
        subfield_closure_contains(a, x + y)
    }
}

/// The generated subfield predicate is closed under negation.
theorem subfield_closure_contains_neg[F: Field](a: Set[F], x: F) {
    subfield_closure_contains(a, x) implies subfield_closure_contains(a, -x)
} by {
    if subfield_closure_contains(a, x) {
        forall(s: Subfield[F]) {
            if set_subset_subfield(a, s) {
                (subfield_closure_contains(a, x) and set_subset_subfield(a, s))
                subfield_closure_contains_in_subfield(a, s, x)
                s.contains(x)
                subfield_contains_neg(s, x)
                s.contains(-x)
            }
        }
        subfield_closure_contains_intro(a, -x)
    }
}

/// The generated subfield predicate is closed under multiplication.
theorem subfield_closure_contains_mul[F: Field](a: Set[F], x: F, y: F) {
    subfield_closure_contains(a, x) and subfield_closure_contains(a, y)
    implies subfield_closure_contains(a, x * y)
} by {
    if subfield_closure_contains(a, x) and subfield_closure_contains(a, y) {
        forall(s: Subfield[F]) {
            if set_subset_subfield(a, s) {
                (subfield_closure_contains(a, x) and set_subset_subfield(a, s))
                subfield_closure_contains_in_subfield(a, s, x)
                s.contains(x)
                (subfield_closure_contains(a, y) and set_subset_subfield(a, s))
                subfield_closure_contains_in_subfield(a, s, y)
                s.contains(y)
                subfield_contains_mul(s, x, y)
                s.contains(x * y)
            }
        }
        forall(s: Subfield[F]) {
            set_subset_subfield(a, s) implies s.contains(x * y)
        }
        subfield_closure_contains(a, x * y) = forall(s: Subfield[F]) {
            set_subset_subfield(a, s) implies s.contains(x * y)
        }
        subfield_closure_contains(a, x * y)
    }
}

/// The generated subfield predicate is closed under inverse of a nonzero element.
theorem subfield_closure_contains_inverse[F: Field](a: Set[F], x: F) {
    subfield_closure_contains(a, x) and x != F.0 implies subfield_closure_contains(a, x.inverse)
} by {
    if subfield_closure_contains(a, x) and x != F.0 {
        forall(s: Subfield[F]) {
            if set_subset_subfield(a, s) {
                (subfield_closure_contains(a, x) and set_subset_subfield(a, s))
                subfield_closure_contains_in_subfield(a, s, x)
                s.contains(x)
                subfield_contains_inverse(s, x)
                s.contains(x.inverse)
            }
        }
        forall(s: Subfield[F]) {
            set_subset_subfield(a, s) implies s.contains(x.inverse)
        }
        subfield_closure_contains(a, x.inverse) = forall(s: Subfield[F]) {
            set_subset_subfield(a, s) implies s.contains(x.inverse)
        }
        subfield_closure_contains(a, x.inverse)
    }
}

/// The least-subfield membership predicate satisfies the subfield constraints.
theorem subfield_closure_contains_is_subfield[F: Field](a: Set[F]) {
    subfield_constraint(subfield_closure_contains(a))
} by {
    let contains: F -> Bool = subfield_closure_contains(a)
    forall(x: F) {
        contains(x) = subfield_closure_contains(a, x)
    }
    subfield_closure_contains_zero(a)
    contains(F.0)
    subfield_zero_constraint(contains)
    subfield_closure_contains_one(a)
    contains(F.1)
    subfield_one_constraint(contains)
    forall(x: F, y: F) {
        if contains(x) and contains(y) {
            subfield_closure_contains(a, x)
            subfield_closure_contains(a, y)
            subfield_closure_contains_add(a, x, y)
            subfield_closure_contains(a, x + y)
            contains(x + y)
        }
    }
    subfield_add_constraint(contains)
    forall(x: F) {
        if contains(x) {
            subfield_closure_contains(a, x)
            subfield_closure_contains_neg(a, x)
            subfield_closure_contains(a, -x)
            contains(-x)
        }
    }
    subfield_neg_constraint(contains)
    subfield_additive_constraint(contains)
    forall(x: F, y: F) {
        if contains(x) and contains(y) {
            subfield_closure_contains(a, x)
            subfield_closure_contains(a, y)
            subfield_closure_contains_mul(a, x, y)
            subfield_closure_contains(a, x * y)
            contains(x * y)
        }
    }
    subfield_mul_constraint(contains)
    forall(x: F) {
        if contains(x) and x != F.0 {
            subfield_closure_contains(a, x)
            subfield_closure_contains_inverse(a, x)
            subfield_closure_contains(a, x.inverse)
            contains(x.inverse)
        }
    }
    subfield_inverse_constraint(contains)
    subfield_multiplicative_constraint(contains)
}

/// The subfield generated by a set of field elements.
let subfield_closure[F: Field](a: Set[F]) -> result: Subfield[F] satisfy {
    Subfield.new(subfield_closure_contains(a)) = Option.some(result)
} by {
    subfield_closure_contains_is_subfield(a)
}

/// Membership in the generated subfield is the least-subfield predicate.
theorem subfield_closure_contains_eq[F: Field](a: Set[F], x: F) {
    subfield_closure(a).contains(x) = subfield_closure_contains(a, x)
} by {
    subfield_closure(a).contains = subfield_closure_contains(a)
}

/// Every generator lies in the generated subfield.
theorem subfield_closure_contains_generator[F: Field](a: Set[F], x: F) {
    a.contains(x) implies subfield_closure(a).contains(x)
} by {
    if a.contains(x) {
        forall(s: Subfield[F]) {
            if set_subset_subfield(a, s) {
                set_subset_subfield_contains(a, s, x)
                s.contains(x)
            }
        }
        subfield_closure_contains_intro(a, x)
        subfield_closure_contains(a, x)
        subfield_closure_contains_eq(a, x)
        subfield_closure(a).contains(x)
    }
}

/// The generated subfield is contained in every subfield containing the generators.
theorem subfield_closure_minimal[F: Field](a: Set[F], s: Subfield[F]) {
    set_subset_subfield(a, s) implies subfield_le(subfield_closure(a), s)
} by {
    if set_subset_subfield(a, s) {
        forall(x: F) {
            if subfield_closure(a).contains(x) {
                subfield_closure_contains_eq(a, x)
                subfield_closure_contains(a, x)
                (subfield_closure_contains(a, x) and set_subset_subfield(a, s))
                subfield_closure_contains_in_subfield(a, s, x)
                s.contains(x)
            }
        }
        subfield_le_intro(subfield_closure(a), s)
    }
}

/// A subfield contains the generated subfield exactly when it contains the generators.
theorem subfield_closure_le_iff_set_subset_subfield[F: Field](a: Set[F], s: Subfield[F]) {
    subfield_le(subfield_closure(a), s) = set_subset_subfield(a, s)
} by {
    if subfield_le(subfield_closure(a), s) {
        subfield_le(subfield_closure(a), s) = forall(x: F) {
            subfield_closure(a).contains(x) implies s.contains(x)
        }
        forall(x: F) {
            if a.contains(x) {
                subfield_closure_contains_generator(a, x)
                subfield_closure(a).contains(x)
                s.contains(x)
            }
        }
        set_subset_subfield_intro(a, s)
    }
    if set_subset_subfield(a, s) {
        subfield_closure_minimal(a, s)
        subfield_le(subfield_closure(a), s)
    }
}

/// The generated subfield is monotone in the generating set.
theorem subfield_closure_mono[F: Field](a: Set[F], b: Set[F]) {
    a.subset(b) implies subfield_le(subfield_closure(a), subfield_closure(b))
} by {
    if a.subset(b) {
        forall(x: F) {
            if a.contains(x) {
                subset_contains(a, b, x)
                b.contains(x)
                subfield_closure_contains_generator(b, x)
                subfield_closure(b).contains(x)
            }
        }
        set_subset_subfield_intro(a, subfield_closure(b))
        set_subset_subfield(a, subfield_closure(b))
        subfield_closure_minimal(a, subfield_closure(b))
        subfield_le(subfield_closure(a), subfield_closure(b))
    }
}

/// Mutually contained generating sets have the same generated subfield.
theorem subfield_closure_eq_of_mutual_subset[F: Field](a: Set[F], b: Set[F]) {
    a.subset(b) and b.subset(a) implies subfield_closure(a) = subfield_closure(b)
} by {
    if a.subset(b) and b.subset(a) {
        subfield_closure_mono(a, b)
        subfield_le(subfield_closure(a), subfield_closure(b))
        subfield_closure_mono(b, a)
        subfield_le(subfield_closure(b), subfield_closure(a))
        subfield_le_antisymm(subfield_closure(a), subfield_closure(b))
    }
}

/// The generated subfield of a subfield's carrier is the original subfield.
theorem subfield_closure_of_carrier[F: Field](s: Subfield[F]) {
    subfield_closure(subfield_carrier(s)) = s
} by {
    set_subset_subfield_carrier_self(s)
    subfield_closure_minimal(subfield_carrier(s), s)
    subfield_le(subfield_closure(subfield_carrier(s)), s)
    forall(x: F) {
        if s.contains(x) {
            subfield_carrier_contains_eq(s, x)
            subfield_carrier(s).contains(x)
            subfield_closure_contains_generator(subfield_carrier(s), x)
            subfield_closure(subfield_carrier(s)).contains(x)
        }
    }
    subfield_le_intro(s, subfield_closure(subfield_carrier(s)))
    subfield_le(s, subfield_closure(subfield_carrier(s)))
    subfield_le_antisymm(subfield_closure(subfield_carrier(s)), s)
}

/// Taking generated subfield closure is idempotent.
theorem subfield_closure_idempotent[F: Field](a: Set[F]) {
    subfield_closure(subfield_carrier(subfield_closure(a))) = subfield_closure(a)
} by {
    subfield_closure_of_carrier(subfield_closure(a))
}
