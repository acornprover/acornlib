/// Basic idempotent elements for multiplicative algebra.

from algebra.monoid.monoid import Monoid
from algebra.mul import Mul

/// An element is idempotent when multiplying it by itself gives itself.
define is_idempotent_elem[M: Mul](a: M) -> Bool {
    a * a = a
}

/// Idempotent-element membership unfolds to the defining square equality.
theorem is_idempotent_elem_iff[M: Mul](a: M) {
    is_idempotent_elem(a) = (a * a = a)
}

/// An idempotent element satisfies the defining square equality.
theorem is_idempotent_elem_eq[M: Mul](a: M) {
    is_idempotent_elem(a) implies a * a = a
} by {
    if is_idempotent_elem(a) {
        is_idempotent_elem(a) = (a * a = a)
        a * a = a
    }
}

/// The defining square equality introduces idempotence.
theorem is_idempotent_elem_intro[M: Mul](a: M) {
    a * a = a implies is_idempotent_elem(a)
} by {
    if a * a = a {
        is_idempotent_elem(a) = (a * a = a)
        is_idempotent_elem(a)
    }
}

/// The unit of a monoid is idempotent.
theorem is_idempotent_elem_one[M: Monoid] {
    is_idempotent_elem(M.1)
} by {
    M.1 * M.1 = M.1
    is_idempotent_elem_intro[M](M.1)
    is_idempotent_elem(M.1)
}
