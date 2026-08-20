from data.basic.set import Set, set_ext, universal_set_contains_eq, set_preimage,
    set_preimage_contains_eq, preimage_contains, intersection_contains_eq
from pair import Pair
from analysis import TopologicalSpace, big_union, generated_open, generated_open_inter,
    generated_open_big_union, is_box, is_box_is_topological_basis, is_open_in_product,
    is_open_in_product_imp_generated_open, generated_open_imp_is_open_in_product,
    product_open_empty, product_open_universal, box, box_contains_eq,
    basis_member_is_open, is_continuous, open_universal, box_is_open_in_product

/// The intersection of two product-open sets is product-open.
theorem product_open_inter[X: TopologicalSpace, Y: TopologicalSpace](
    s: Set[Pair[X, Y]], t: Set[Pair[X, Y]]) {
    is_open_in_product[X, Y](s) and is_open_in_product[X, Y](t)
        implies is_open_in_product[X, Y](s.intersection(t))
} by {
    if is_open_in_product[X, Y](s) and is_open_in_product[X, Y](t) {
        is_open_in_product_imp_generated_open[X, Y](s)
        is_open_in_product_imp_generated_open[X, Y](t)
        is_box_is_topological_basis[X, Y]
        let b: Set[Pair[X, Y]] -> Bool = is_box[X, Y]
        generated_open(b, s)
        generated_open(b, t)
        generated_open_inter[Pair[X, Y]](b, s, t)
        generated_open(b, s.intersection(t))
        generated_open(is_box[X, Y], s.intersection(t))
        generated_open_imp_is_open_in_product[X, Y](s.intersection(t))
        is_open_in_product[X, Y](s.intersection(t))
    }
}

/// The union of any family of product-open sets is product-open.
theorem product_open_big_union[X: TopologicalSpace, Y: TopologicalSpace](
    c: Set[Pair[X, Y]] -> Bool) {
    (forall(s: Set[Pair[X, Y]]) { c(s) implies is_open_in_product[X, Y](s) })
        implies is_open_in_product[X, Y](big_union(c))
} by {
    if forall(s: Set[Pair[X, Y]]) { c(s) implies is_open_in_product[X, Y](s) } {
        let b: Set[Pair[X, Y]] -> Bool = is_box[X, Y]
        forall(s: Set[Pair[X, Y]]) {
            if c(s) {
                is_open_in_product[X, Y](s)
                is_open_in_product_imp_generated_open[X, Y](s)
                generated_open(is_box[X, Y], s)
                generated_open(b, s)
            }
        }
        generated_open_big_union[Pair[X, Y]](b, c)
        generated_open(b, big_union(c))
        generated_open(is_box[X, Y], big_union(c))
        generated_open_imp_is_open_in_product[X, Y](big_union(c))
        is_open_in_product[X, Y](big_union(c))
    }
}

/// The product topology on `Pair[X, Y]`: open sets are the product-open sets generated
/// by the boxes `box(u, v)` for opens `u`, `v` of the factors.
instance Pair[X: TopologicalSpace, Y: TopologicalSpace]: TopologicalSpace {
    let is_open: Set[Pair[X, Y]] -> Bool = is_open_in_product[X, Y]
}

/// Openness in the product topology coincides with `is_open_in_product`.
theorem pair_is_open_eq[X: TopologicalSpace, Y: TopologicalSpace](
    s: Set[Pair[X, Y]]) {
    Pair[X, Y].is_open(s) = is_open_in_product[X, Y](s)
}

/// A box of open factors is open in the carrier product topology on `Pair[X, Y]`.
theorem box_open[X: TopologicalSpace, Y: TopologicalSpace](u: Set[X], v: Set[Y]) {
    X.is_open(u) and Y.is_open(v) implies Pair[X, Y].is_open(box(u, v))
} by {
    if X.is_open(u) and Y.is_open(v) {
        box_is_open_in_product[X, Y](u, v)
        pair_is_open_eq[X, Y](box(u, v))
    }
}

/// The left projection of a pair onto its first factor.
define pair_fst[X, Y](p: Pair[X, Y]) -> X {
    p.first
}

/// The right projection of a pair onto its second factor.
define pair_snd[X, Y](p: Pair[X, Y]) -> Y {
    p.second
}

/// The preimage of a factor set under the left projection is its box with the whole right factor.
theorem pair_fst_preimage_eq[X, Y](u: Set[X]) {
    set_preimage(pair_fst[X, Y], u) = box(u, Set[Y].universal_set)
} by {
    forall(p: Pair[X, Y]) {
        set_preimage_contains_eq(pair_fst[X, Y], u, p)
        pair_fst[X, Y](p) = p.first
        preimage_contains(pair_fst[X, Y], u, p) = u.contains(p.first)
        box_contains_eq(u, Set[Y].universal_set, p)
        universal_set_contains_eq[Y](p.second)
        set_preimage(pair_fst[X, Y], u).contains(p) = box(u, Set[Y].universal_set).contains(p)
    }
    set_ext(set_preimage(pair_fst[X, Y], u), box(u, Set[Y].universal_set))
}

/// The preimage of a factor set under the right projection is the whole left factor boxed with it.
theorem pair_snd_preimage_eq[X, Y](v: Set[Y]) {
    set_preimage(pair_snd[X, Y], v) = box(Set[X].universal_set, v)
} by {
    forall(p: Pair[X, Y]) {
        set_preimage_contains_eq(pair_snd[X, Y], v, p)
        pair_snd[X, Y](p) = p.second
        preimage_contains(pair_snd[X, Y], v, p) = v.contains(p.second)
        box_contains_eq(Set[X].universal_set, v, p)
        universal_set_contains_eq[X](p.first)
        set_preimage(pair_snd[X, Y], v).contains(p) = box(Set[X].universal_set, v).contains(p)
    }
    set_ext(set_preimage(pair_snd[X, Y], v), box(Set[X].universal_set, v))
}

/// A box is the intersection of the two projection preimages of its factors.
theorem box_eq_inter_preimages[X, Y](u: Set[X], v: Set[Y]) {
    box(u, v) = set_preimage(pair_fst[X, Y], u).intersection(set_preimage(pair_snd[X, Y], v))
} by {
    let l = set_preimage(pair_fst[X, Y], u)
    let r = set_preimage(pair_snd[X, Y], v)
    forall(p: Pair[X, Y]) {
        box_contains_eq(u, v, p)
        set_preimage_contains_eq(pair_fst[X, Y], u, p)
        pair_fst[X, Y](p) = p.first
        preimage_contains(pair_fst[X, Y], u, p) = u.contains(p.first)
        set_preimage_contains_eq(pair_snd[X, Y], v, p)
        pair_snd[X, Y](p) = p.second
        preimage_contains(pair_snd[X, Y], v, p) = v.contains(p.second)
        intersection_contains_eq(l, r, p)
        box(u, v).contains(p) = l.intersection(r).contains(p)
    }
    set_ext(box(u, v), l.intersection(r))
}

/// The left projection is continuous from the product topology to the first factor.
theorem pair_fst_continuous[X: TopologicalSpace, Y: TopologicalSpace] {
    is_continuous(pair_fst[X, Y])
} by {
    forall(u: Set[X]) {
        if X.is_open(u) {
            open_universal[Y]
            box_open[X, Y](u, Set[Y].universal_set)
            Pair[X, Y].is_open(box(u, Set[Y].universal_set))
            pair_fst_preimage_eq[X, Y](u)
            Pair[X, Y].is_open(set_preimage(pair_fst[X, Y], u))
        }
    }
    is_continuous(pair_fst[X, Y]) = forall(u: Set[X]) {
        X.is_open(u) implies Pair[X, Y].is_open(set_preimage(pair_fst[X, Y], u))
    }
}

/// The right projection is continuous from the product topology to the second factor.
theorem pair_snd_continuous[X: TopologicalSpace, Y: TopologicalSpace] {
    is_continuous(pair_snd[X, Y])
} by {
    forall(v: Set[Y]) {
        if Y.is_open(v) {
            open_universal[X]
            box_open[X, Y](Set[X].universal_set, v)
            Pair[X, Y].is_open(box(Set[X].universal_set, v))
            pair_snd_preimage_eq[X, Y](v)
            Pair[X, Y].is_open(set_preimage(pair_snd[X, Y], v))
        }
    }
    is_continuous(pair_snd[X, Y]) = forall(v: Set[Y]) {
        Y.is_open(v) implies Pair[X, Y].is_open(set_preimage(pair_snd[X, Y], v))
    }
}
