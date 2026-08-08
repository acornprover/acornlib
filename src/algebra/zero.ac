/// The "Zero" typeclass represents anything that has an additive identity element.
typeclass A: Zero {
    0: A
}
