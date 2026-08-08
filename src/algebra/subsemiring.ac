from semiring import Semiring
from data.basic.set import Set
from data.basic.functions import predicate_extensionality

/// True if a subset contains the additive identity.
define subsemiring_zero_constraint[S: Semiring](contains: S -> Bool) -> Bool {
    contains(S.0)
}

/// True if a subset contains the multiplicative identity.
define subsemiring_one_constraint[S: Semiring](contains: S -> Bool) -> Bool {
    contains(S.1)
}

/// True if a subset is closed under addition.
define subsemiring_add_constraint[S: Semiring](contains: S -> Bool) -> Bool {
    forall(a: S, b: S) {
        contains(a) and contains(b) implies contains(a + b)
    }
}

/// True if a subset is closed under multiplication.
define subsemiring_mul_constraint[S: Semiring](contains: S -> Bool) -> Bool {
    forall(a: S, b: S) {
        contains(a) and contains(b) implies contains(a * b)
    }
}

/// True if a subset satisfies all the requirements to be a subsemiring.
define subsemiring_constraint[S: Semiring](contains: S -> Bool) -> Bool {
    subsemiring_zero_constraint(contains) and subsemiring_one_constraint(contains) and
        subsemiring_add_constraint(contains) and subsemiring_mul_constraint(contains)
}

/// A subsemiring of a semiring, represented as a subset containing zero and one and closed under addition and multiplication.
structure Subsemiring[S: Semiring] {
    /// True if the given element is a member of this subsemiring.
    contains: S -> Bool
} constraint {
    subsemiring_constraint(contains)
}

/// Every subsemiring's membership predicate satisfies the subsemiring constraint.
theorem subsemiring_satisfies_constraint[S: Semiring](s: Subsemiring[S]) {
    subsemiring_constraint(s.contains)
}

/// Every subsemiring's membership predicate satisfies the zero constraint.
theorem subsemiring_satisfies_zero[S: Semiring](s: Subsemiring[S]) {
    subsemiring_zero_constraint(s.contains)
} by {
    subsemiring_satisfies_constraint(s)
    subsemiring_constraint(s.contains) =
        (subsemiring_zero_constraint(s.contains) and subsemiring_one_constraint(s.contains) and
            subsemiring_add_constraint(s.contains) and subsemiring_mul_constraint(s.contains))
}

/// Every subsemiring's membership predicate satisfies the one constraint.
theorem subsemiring_satisfies_one[S: Semiring](s: Subsemiring[S]) {
    subsemiring_one_constraint(s.contains)
} by {
    subsemiring_satisfies_constraint(s)
    subsemiring_constraint(s.contains) =
        (subsemiring_zero_constraint(s.contains) and subsemiring_one_constraint(s.contains) and
            subsemiring_add_constraint(s.contains) and subsemiring_mul_constraint(s.contains))
}

/// Every subsemiring's membership predicate satisfies the add constraint.
theorem subsemiring_satisfies_add[S: Semiring](s: Subsemiring[S]) {
    forall(a: S, b: S) {
        s.contains(a) and s.contains(b) implies s.contains(a + b)
    }
} by {
    subsemiring_satisfies_constraint(s)
    subsemiring_constraint(s.contains) =
        (subsemiring_zero_constraint(s.contains) and subsemiring_one_constraint(s.contains) and
            subsemiring_add_constraint(s.contains) and subsemiring_mul_constraint(s.contains))
    subsemiring_add_constraint(s.contains)
    subsemiring_add_constraint(s.contains) = forall(a: S, b: S) {
        s.contains(a) and s.contains(b) implies s.contains(a + b)
    }
}

/// Every subsemiring's membership predicate satisfies the mul constraint.
theorem subsemiring_satisfies_mul[S: Semiring](s: Subsemiring[S]) {
    forall(a: S, b: S) {
        s.contains(a) and s.contains(b) implies s.contains(a * b)
    }
} by {
    subsemiring_satisfies_constraint(s)
    subsemiring_constraint(s.contains) =
        (subsemiring_zero_constraint(s.contains) and subsemiring_one_constraint(s.contains) and
            subsemiring_add_constraint(s.contains) and subsemiring_mul_constraint(s.contains))
    subsemiring_mul_constraint(s.contains)
    subsemiring_mul_constraint(s.contains) = forall(a: S, b: S) {
        s.contains(a) and s.contains(b) implies s.contains(a * b)
    }
}

/// Subsemiring extensionality from equality of membership.
theorem subsemiring_ext[S: Semiring](a: Subsemiring[S], b: Subsemiring[S]) {
    (forall(x: S) { a.contains(x) = b.contains(x) }) implies a = b
} by {
    if forall(x: S) { a.contains(x) = b.contains(x) } {
        predicate_extensionality(a.contains, b.contains)
        a.contains = b.contains
        a = b
    }
}

/// Equal subsemirings have equal membership predicates.
theorem subsemiring_eq_contains[S: Semiring](a: Subsemiring[S], b: Subsemiring[S]) {
    a = b implies a.contains = b.contains
}

/// Equal subsemirings have equal membership at every element.
theorem subsemiring_eq_contains_at[S: Semiring](a: Subsemiring[S], b: Subsemiring[S], x: S) {
    a = b implies a.contains(x) = b.contains(x)
} by {
    if a = b {
        a.contains = b.contains
        a.contains(x) = b.contains(x)
    }
}

/// Equality of membership predicates determines equality of subsemirings.
theorem subsemiring_eq_of_contains_eq[S: Semiring](a: Subsemiring[S], b: Subsemiring[S]) {
    a.contains = b.contains implies a = b
} by {
    if a.contains = b.contains {
        forall(x: S) {
            a.contains(x) = b.contains(x)
        }
        subsemiring_ext(a, b)
    }
}

/// Pointwise equality of membership determines equality of subsemirings.
theorem subsemiring_eq_of_contains_at_eq[S: Semiring](a: Subsemiring[S], b: Subsemiring[S]) {
    (forall(x: S) { a.contains(x) = b.contains(x) }) implies a = b
} by {
    if forall(x: S) { a.contains(x) = b.contains(x) } {
        subsemiring_ext(a, b)
    }
}

/// Membership of zero in any subsemiring.
theorem subsemiring_contains_zero[S: Semiring](s: Subsemiring[S]) {
    s.contains(S.0)
} by {
    subsemiring_satisfies_zero(s)
    subsemiring_zero_constraint(s.contains) = s.contains(S.0)
}

/// Membership of one in any subsemiring.
theorem subsemiring_contains_one[S: Semiring](s: Subsemiring[S]) {
    s.contains(S.1)
} by {
    subsemiring_satisfies_one(s)
    subsemiring_one_constraint(s.contains) = s.contains(S.1)
}

/// Closure of any subsemiring under addition.
theorem subsemiring_contains_add[S: Semiring](s: Subsemiring[S], a: S, b: S) {
    s.contains(a) and s.contains(b) implies s.contains(a + b)
} by {
    if s.contains(a) and s.contains(b) {
        subsemiring_satisfies_add(s)
        s.contains(a + b)
    }
}

/// Closure of any subsemiring under multiplication.
theorem subsemiring_contains_mul[S: Semiring](s: Subsemiring[S], a: S, b: S) {
    s.contains(a) and s.contains(b) implies s.contains(a * b)
} by {
    if s.contains(a) and s.contains(b) {
        subsemiring_satisfies_mul(s)
        s.contains(a * b)
    }
}

/// The common membership predicate of two subsemirings.
define subsemiring_intersection_contains[S: Semiring](a: Subsemiring[S], b: Subsemiring[S], x: S) -> Bool {
    a.contains(x) and b.contains(x)
}

/// The common part of two subsemirings contains zero.
theorem subsemiring_intersection_zero_constraint[S: Semiring](a: Subsemiring[S], b: Subsemiring[S]) {
    subsemiring_zero_constraint(subsemiring_intersection_contains(a, b))
} by {
    subsemiring_contains_zero(a)
    subsemiring_contains_zero(b)
    subsemiring_intersection_contains(a, b, S.0)
}

/// The common part of two subsemirings contains one.
theorem subsemiring_intersection_one_constraint[S: Semiring](a: Subsemiring[S], b: Subsemiring[S]) {
    subsemiring_one_constraint(subsemiring_intersection_contains(a, b))
} by {
    subsemiring_contains_one(a)
    subsemiring_contains_one(b)
    subsemiring_intersection_contains(a, b, S.1)
}

/// The common part of two subsemirings is closed under addition.
theorem subsemiring_intersection_add_constraint[S: Semiring](a: Subsemiring[S], b: Subsemiring[S]) {
    subsemiring_add_constraint(subsemiring_intersection_contains(a, b))
} by {
    forall(x: S, y: S) {
        if subsemiring_intersection_contains(a, b, x) and subsemiring_intersection_contains(a, b, y) {
            a.contains(x)
            b.contains(x)
            a.contains(y)
            b.contains(y)
            subsemiring_contains_add(a, x, y)
            subsemiring_contains_add(b, x, y)
            a.contains(x + y)
            b.contains(x + y)
            subsemiring_intersection_contains(a, b, x + y)
        }
    }
}

/// The common part of two subsemirings is closed under multiplication.
theorem subsemiring_intersection_mul_constraint[S: Semiring](a: Subsemiring[S], b: Subsemiring[S]) {
    subsemiring_mul_constraint(subsemiring_intersection_contains(a, b))
} by {
    forall(x: S, y: S) {
        if subsemiring_intersection_contains(a, b, x) and subsemiring_intersection_contains(a, b, y) {
            a.contains(x)
            b.contains(x)
            a.contains(y)
            b.contains(y)
            subsemiring_contains_mul(a, x, y)
            subsemiring_contains_mul(b, x, y)
            a.contains(x * y)
            b.contains(x * y)
            subsemiring_intersection_contains(a, b, x * y)
        }
    }
}

/// The common part of two subsemirings is a subsemiring.
theorem subsemiring_intersection_constraint[S: Semiring](a: Subsemiring[S], b: Subsemiring[S]) {
    subsemiring_constraint(subsemiring_intersection_contains(a, b))
} by {
    subsemiring_intersection_zero_constraint(a, b)
    subsemiring_intersection_one_constraint(a, b)
    subsemiring_intersection_add_constraint(a, b)
    subsemiring_intersection_mul_constraint(a, b)
}

/// The common part of two subsemirings.
let subsemiring_intersection[S: Semiring](a: Subsemiring[S], b: Subsemiring[S]) -> result: Subsemiring[S] satisfy {
    Subsemiring.new(subsemiring_intersection_contains(a, b)) = Option.some(result)
} by {
    subsemiring_intersection_constraint(a, b)
}

attributes Subsemiring[S: Semiring] {
    /// Subsemiring extensionality from pointwise equality of membership.
    let ext = subsemiring_ext[S]

    /// The subset of semiring elements belonging to this subsemiring.
    define as_set(self) -> Set[S] {
        Set[S].new(self.contains)
    }

    /// The common part of two subsemirings.
    let intersection: (Subsemiring[S], Subsemiring[S]) -> Subsemiring[S] = subsemiring_intersection
}

/// Membership in the underlying set is membership in the subsemiring.
theorem subsemiring_as_set_contains_eq[S: Semiring](s: Subsemiring[S], x: S) {
    s.as_set.contains(x) = s.contains(x)
} by {
    s.as_set.contains(x) = s.contains(x)
}

/// Membership in a subsemiring is membership in its underlying set.
theorem subsemiring_contains_as_set_eq[S: Semiring](s: Subsemiring[S], x: S) {
    s.contains(x) = s.as_set.contains(x)
} by {
    subsemiring_as_set_contains_eq(s, x)
    s.contains(x) = s.as_set.contains(x)
}

/// Membership in the intersection means membership in both subsemirings.
theorem subsemiring_intersection_contains_eq[S: Semiring](a: Subsemiring[S], b: Subsemiring[S], x: S) {
    a.intersection(b).contains(x) = (a.contains(x) and b.contains(x))
} by {
    Subsemiring.new(subsemiring_intersection_contains(a, b)) = Option.some(a.intersection(b))
    a.intersection(b).contains = subsemiring_intersection_contains(a, b)
    a.intersection(b).contains(x) = subsemiring_intersection_contains(a, b, x)
    subsemiring_intersection_contains(a, b, x) = (a.contains(x) and b.contains(x))
}

/// The intersection is contained in the left subsemiring at each element.
theorem subsemiring_intersection_subset_left[S: Semiring](a: Subsemiring[S], b: Subsemiring[S], x: S) {
    a.intersection(b).contains(x) implies a.contains(x)
} by {
    if a.intersection(b).contains(x) {
        subsemiring_intersection_contains_eq(a, b, x)
        a.contains(x)
    }
}

/// The intersection is contained in the right subsemiring at each element.
theorem subsemiring_intersection_subset_right[S: Semiring](a: Subsemiring[S], b: Subsemiring[S], x: S) {
    a.intersection(b).contains(x) implies b.contains(x)
} by {
    if a.intersection(b).contains(x) {
        subsemiring_intersection_contains_eq(a, b, x)
        b.contains(x)
    }
}

/// True if one subsemiring is contained in another.
define subsemiring_subset[S: Semiring](a: Subsemiring[S], b: Subsemiring[S]) -> Bool {
    forall(x: S) {
        a.contains(x) implies b.contains(x)
    }
}

/// Subsemiring containment is containment of the underlying sets.
theorem subsemiring_subset_as_set_eq[S: Semiring](a: Subsemiring[S], b: Subsemiring[S]) {
    subsemiring_subset(a, b) = a.as_set.subset(b.as_set)
} by {
    if subsemiring_subset(a, b) {
        a.as_set.subset(b.as_set) = forall(x: S) {
            a.as_set.contains(x) implies b.as_set.contains(x)
        }
        forall(x: S) {
            if a.as_set.contains(x) {
                subsemiring_as_set_contains_eq(a, x)
                a.contains(x)
                b.contains(x)
                subsemiring_as_set_contains_eq(b, x)
                b.as_set.contains(x)
            }
            a.as_set.contains(x) implies b.as_set.contains(x)
        }
        a.as_set.subset(b.as_set)
    }
    if a.as_set.subset(b.as_set) {
        forall(x: S) {
            if a.contains(x) {
                subsemiring_as_set_contains_eq(a, x)
                a.as_set.contains(x)
                a.as_set.subset(b.as_set) = forall(y: S) {
                    a.as_set.contains(y) implies b.as_set.contains(y)
                }
                b.as_set.contains(x)
                subsemiring_as_set_contains_eq(b, x)
                b.contains(x)
            }
        }
        subsemiring_subset(a, b)
    }
    subsemiring_subset(a, b) = a.as_set.subset(b.as_set)
}

/// Subsemiring inclusion is reflexive.
theorem subsemiring_subset_refl[S: Semiring](a: Subsemiring[S]) {
    subsemiring_subset(a, a)
} by {
    forall(x: S) {
        a.contains(x) implies a.contains(x)
    }
}

/// Subsemiring inclusion is transitive.
theorem subsemiring_subset_trans[S: Semiring](a: Subsemiring[S], b: Subsemiring[S], c: Subsemiring[S]) {
    subsemiring_subset(a, b) and subsemiring_subset(b, c) implies subsemiring_subset(a, c)
} by {
    if subsemiring_subset(a, b) and subsemiring_subset(b, c) {
        forall(x: S) {
            if a.contains(x) {
                b.contains(x)
                c.contains(x)
            }
        }
    }
}

/// The intersection of two subsemirings is contained in the left subsemiring.
theorem subsemiring_intersection_subset_left_relation[S: Semiring](a: Subsemiring[S], b: Subsemiring[S]) {
    subsemiring_subset(a.intersection(b), a)
} by {
    subsemiring_subset(a.intersection(b), a) = forall(x: S) {
        a.intersection(b).contains(x) implies a.contains(x)
    }
    forall(x: S) {
        if a.intersection(b).contains(x) {
            subsemiring_intersection_contains_eq(a, b, x)
            a.contains(x)
        }
        a.intersection(b).contains(x) implies a.contains(x)
    }
}

/// The intersection of two subsemirings is contained in the right subsemiring.
theorem subsemiring_intersection_subset_right_relation[S: Semiring](a: Subsemiring[S], b: Subsemiring[S]) {
    subsemiring_subset(a.intersection(b), b)
} by {
    subsemiring_subset(a.intersection(b), b) = forall(x: S) {
        a.intersection(b).contains(x) implies b.contains(x)
    }
    forall(x: S) {
        if a.intersection(b).contains(x) {
            subsemiring_intersection_subset_right(a, b, x)
            b.contains(x)
        }
        a.intersection(b).contains(x) implies b.contains(x)
    }
}

/// Every common subsemiring of two subsemirings is contained in their intersection.
theorem subsemiring_subset_intersection_of_subset_left_right[S: Semiring](
    c: Subsemiring[S],
    a: Subsemiring[S],
    b: Subsemiring[S]
) {
    subsemiring_subset(c, a) and subsemiring_subset(c, b) implies subsemiring_subset(c, a.intersection(b))
} by {
    if subsemiring_subset(c, a) and subsemiring_subset(c, b) {
        forall(x: S) {
            if c.contains(x) {
                a.contains(x)
                b.contains(x)
                subsemiring_intersection_contains_eq(a, b, x)
                a.intersection(b).contains(x)
            }
        }
    }
}

/// Containment in an intersection is equivalent to containment in both subsemirings.
theorem subsemiring_subset_intersection_iff[S: Semiring](c: Subsemiring[S], a: Subsemiring[S], b: Subsemiring[S]) {
    subsemiring_subset(c, a.intersection(b)) = (subsemiring_subset(c, a) and subsemiring_subset(c, b))
} by {
    if subsemiring_subset(c, a.intersection(b)) {
        subsemiring_intersection_subset_left_relation(a, b)
        subsemiring_subset_trans(c, a.intersection(b), a)
        subsemiring_subset(c, a)
        subsemiring_intersection_subset_right_relation(a, b)
        subsemiring_subset_trans(c, a.intersection(b), b)
        subsemiring_subset(c, b)
        subsemiring_subset(c, a) and subsemiring_subset(c, b)
    }
    if subsemiring_subset(c, a) and subsemiring_subset(c, b) {
        subsemiring_subset_intersection_of_subset_left_right(c, a, b)
        subsemiring_subset(c, a.intersection(b))
    }
    subsemiring_subset(c, a.intersection(b)) = (subsemiring_subset(c, a) and subsemiring_subset(c, b))
}

/// Mutual inclusion of subsemirings forces equality.
theorem subsemiring_subset_antisymm[S: Semiring](a: Subsemiring[S], b: Subsemiring[S]) {
    subsemiring_subset(a, b) and subsemiring_subset(b, a) implies a = b
} by {
    if subsemiring_subset(a, b) and subsemiring_subset(b, a) {
        subsemiring_subset(a, b) = forall(y: S) {
            a.contains(y) implies b.contains(y)
        }
        subsemiring_subset(b, a) = forall(y: S) {
            b.contains(y) implies a.contains(y)
        }
        forall(x: S) {
            if a.contains(x) {
                b.contains(x)
            }
            if b.contains(x) {
                a.contains(x)
            }
            a.contains(x) = b.contains(x)
        }
        predicate_extensionality(a.contains, b.contains)
        a.contains = b.contains
        subsemiring_eq_of_contains_eq(a, b)
        a = b
    }
}

/// Equality of subsemirings is equivalent to mutual inclusion.
theorem subsemiring_eq_iff_subset_both[S: Semiring](a: Subsemiring[S], b: Subsemiring[S]) {
    a = b = (subsemiring_subset(a, b) and subsemiring_subset(b, a))
} by {
    if a = b {
        subsemiring_subset_refl(a)
        subsemiring_subset(a, b)
        subsemiring_subset(b, a)
        subsemiring_subset(a, b) and subsemiring_subset(b, a)
    }
    if subsemiring_subset(a, b) and subsemiring_subset(b, a) {
        subsemiring_subset_antisymm(a, b)
        a = b
    }
    a = b = (subsemiring_subset(a, b) and subsemiring_subset(b, a))
}

/// Subsemiring inclusion transports membership from the smaller subsemiring to the larger one.
theorem subsemiring_subset_contains[S: Semiring](a: Subsemiring[S], b: Subsemiring[S], x: S) {
    subsemiring_subset(a, b) and a.contains(x) implies b.contains(x)
} by {
    if subsemiring_subset(a, b) and a.contains(x) {
        subsemiring_subset(a, b) = forall(y: S) {
            a.contains(y) implies b.contains(y)
        }
        b.contains(x)
    }
}

/// Elementwise membership implication gives subsemiring inclusion.
theorem subsemiring_subset_of_contains[S: Semiring](a: Subsemiring[S], b: Subsemiring[S]) {
    (forall(x: S) { a.contains(x) implies b.contains(x) }) implies subsemiring_subset(a, b)
} by {
    if forall(x: S) { a.contains(x) implies b.contains(x) } {
        subsemiring_subset(a, b)
    }
}

/// Underlying-set inclusion transports subsemiring membership.
theorem subsemiring_as_set_subset_contains[S: Semiring](a: Subsemiring[S], b: Subsemiring[S], x: S) {
    a.as_set.subset(b.as_set) and a.contains(x) implies b.contains(x)
} by {
    if a.as_set.subset(b.as_set) and a.contains(x) {
        subsemiring_subset_as_set_eq(a, b)
        subsemiring_subset(a, b)
        subsemiring_subset_contains(a, b, x)
    }
}

/// The intersection is the greatest lower bound for subsemiring inclusion.
theorem subsemiring_intersection_greatest_lower_bound[S: Semiring](
    c: Subsemiring[S],
    a: Subsemiring[S],
    b: Subsemiring[S]
) {
    subsemiring_subset(c, a.intersection(b)) = (subsemiring_subset(c, a) and subsemiring_subset(c, b))
} by {
    subsemiring_subset_intersection_iff(c, a, b)
}

/// Intersections of subsemirings are monotone in the left argument.
theorem subsemiring_intersection_mono_left[S: Semiring](
    a: Subsemiring[S],
    b: Subsemiring[S],
    c: Subsemiring[S]
) {
    subsemiring_subset(a, b) implies subsemiring_subset(a.intersection(c), b.intersection(c))
} by {
    if subsemiring_subset(a, b) {
        subsemiring_intersection_subset_left_relation(a, c)
        subsemiring_subset_trans(a.intersection(c), a, b)
        subsemiring_subset(a.intersection(c), b)
        subsemiring_intersection_subset_right_relation(a, c)
        subsemiring_subset(a.intersection(c), c)
        subsemiring_subset_intersection_of_subset_left_right(a.intersection(c), b, c)
        subsemiring_subset(a.intersection(c), b.intersection(c))
    }
}

/// Intersections of subsemirings are monotone in the right argument.
theorem subsemiring_intersection_mono_right[S: Semiring](
    a: Subsemiring[S],
    b: Subsemiring[S],
    c: Subsemiring[S]
) {
    subsemiring_subset(a, b) implies subsemiring_subset(c.intersection(a), c.intersection(b))
} by {
    if subsemiring_subset(a, b) {
        subsemiring_intersection_subset_left_relation(c, a)
        subsemiring_subset(c.intersection(a), c)
        subsemiring_intersection_subset_right_relation(c, a)
        subsemiring_subset_trans(c.intersection(a), a, b)
        subsemiring_subset(c.intersection(a), b)
        subsemiring_subset_intersection_of_subset_left_right(c.intersection(a), c, b)
        subsemiring_subset(c.intersection(a), c.intersection(b))
    }
}

/// Intersections of subsemirings are monotone in both arguments.
theorem subsemiring_intersection_mono[S: Semiring](
    a: Subsemiring[S],
    b: Subsemiring[S],
    c: Subsemiring[S],
    d: Subsemiring[S]
) {
    subsemiring_subset(a, b) and subsemiring_subset(c, d) implies
        subsemiring_subset(a.intersection(c), b.intersection(d))
} by {
    if subsemiring_subset(a, b) and subsemiring_subset(c, d) {
        subsemiring_intersection_mono_left(a, b, c)
        subsemiring_subset(a.intersection(c), b.intersection(c))
        subsemiring_intersection_mono_right(c, d, b)
        subsemiring_subset(b.intersection(c), b.intersection(d))
        subsemiring_subset_trans(a.intersection(c), b.intersection(c), b.intersection(d))
        subsemiring_subset(a.intersection(c), b.intersection(d))
    }
}

/// Mutual inclusion of subsemirings is equivalent to equality.
theorem subsemiring_subset_antisymm_iff_eq[S: Semiring](a: Subsemiring[S], b: Subsemiring[S]) {
    (subsemiring_subset(a, b) and subsemiring_subset(b, a)) = (a = b)
} by {
    if subsemiring_subset(a, b) and subsemiring_subset(b, a) {
        subsemiring_subset_antisymm(a, b)
        a = b
    }
    if a = b {
        subsemiring_subset_refl(a)
        subsemiring_subset(a, b)
        subsemiring_subset(b, a)
    }
    (subsemiring_subset(a, b) and subsemiring_subset(b, a)) = (a = b)
}

/// The full membership predicate is a subsemiring predicate.
define full_subsemiring_contains[S: Semiring](a: S) -> Bool {
    true
}

/// The full subset of any semiring is a subsemiring.
theorem full_subsemiring_constraint[S: Semiring] {
    subsemiring_constraint(full_subsemiring_contains[S])
} by {
    full_subsemiring_contains[S](S.0)
    full_subsemiring_contains[S](S.1)
    subsemiring_zero_constraint(full_subsemiring_contains[S])
    subsemiring_one_constraint(full_subsemiring_contains[S])
    forall(a: S, b: S) {
        if full_subsemiring_contains[S](a) and full_subsemiring_contains[S](b) {
            full_subsemiring_contains[S](a + b)
            full_subsemiring_contains[S](a * b)
        }
    }
    subsemiring_add_constraint(full_subsemiring_contains[S])
    subsemiring_mul_constraint(full_subsemiring_contains[S])
}

/// The full subsemiring, containing every element.
let full_subsemiring[S: Semiring]: Subsemiring[S] satisfy {
    Subsemiring.new(full_subsemiring_contains[S]) = Option.some(full_subsemiring)
}

attributes Subsemiring[S: Semiring] {
    /// The full subsemiring.
    let full = full_subsemiring[S]
}

/// Every element belongs to the full subsemiring.
theorem full_subsemiring_contains_everything[S: Semiring](a: S) {
    full_subsemiring[S].contains(a)
}

/// Membership in the full subsemiring is always true.
theorem full_subsemiring_contains_eq[S: Semiring](a: S) {
    full_subsemiring[S].contains(a) = true
} by {
    full_subsemiring_contains_everything(a)
}

/// Every subsemiring is contained in the full subsemiring.
theorem subsemiring_subset_full[S: Semiring](s: Subsemiring[S]) {
    subsemiring_subset(s, full_subsemiring[S])
} by {
    forall(x: S) {
        if s.contains(x) {
            full_subsemiring_contains_everything(x)
        }
    }
}
