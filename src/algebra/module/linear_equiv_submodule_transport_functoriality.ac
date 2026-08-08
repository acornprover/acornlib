/// Functoriality and lattice laws for transporting submodules across bundled linear equivalences.

from algebra.ring.ring import Ring
from algebra.add_comm_group import AddCommGroup
from algebra.module.module import Module
from data.basic.functions import identity_fn, compose
from algebra.module.module_iso import LinearEquiv, linear_equiv_identity, linear_equiv_identity_src,
    linear_equiv_identity_dst, linear_equiv_symm, linear_equiv_symm_src,
    linear_equiv_symm_dst, linear_equiv_compose, linear_equiv_compose_src,
    linear_equiv_compose_dst, linear_equiv_left_inverse, linear_equiv_right_inverse,
    identity_fn_two_sided_inverse_self, linear_equiv_inv_fn_is_linear_map,
    linear_equiv_inv_map_eq_zero_iff_eq_zero
from algebra.module.submodule import Submodule, submodule_ext, submodule_eq_contains_at,
    submodule_subset, submodule_subset_refl, submodule_subset_antisymm,
    submodule_subset_sup_left,
    submodule_subset_sup_right, submodule_sup, submodule_sup_carrier_eq,
    submodule_sup_subset_iff, submodule_intersection_carrier_eq,
    submodule_intersection_contains_eq_of_carrier_eq, zero_submodule,
    zero_submodule_carrier_eq, zero_submodule_contains_eq, full_submodule,
    full_submodule_carrier_eq, full_submodule_contains_eq,
    linear_map_preimage_submodule_carrier_eq
from algebra.module.module_hom import identity_fn_is_linear_map
from algebra.module.linear_equiv_submodule import linear_equiv_submodule_map,
    linear_equiv_submodule_map_contains_eq, linear_equiv_submodule_map_contains_forward,
    linear_equiv_submodule_map_mono

/// Transporting by the identity equivalence preserves pointwise membership.
theorem linear_equiv_submodule_map_identity_contains_eq[R: Ring, M: AddCommGroup](
    m: Module[R, M], s: Submodule[R, M], x: M
) {
    s.carrier = m implies linear_equiv_submodule_map(linear_equiv_identity(m), s).contains(x) = s.contains(x)
} by {
    if s.carrier = m {
        linear_equiv_identity_src(m)
        linear_equiv_identity(m).src = m
        linear_equiv_submodule_map_contains_eq(linear_equiv_identity(m), s, x)
        linear_equiv_submodule_map(linear_equiv_identity(m), s).contains(x) =
            s.contains(linear_equiv_identity(m).inv_fn(x))
        identity_fn_is_linear_map(m)
        identity_fn_two_sided_inverse_self[M]
        let g: M -> M = identity_fn[M]
        let id_e: LinearEquiv[R, M, M] satisfy {
            LinearEquiv[R, M, M].new(m, m, identity_fn[M], g) = Option.some(id_e)
        }
        id_e.inv_fn = g
        id_e.inv_fn = identity_fn[M]
        linear_equiv_identity(m) = id_e
        linear_equiv_identity(m).inv_fn = identity_fn[M]
        linear_equiv_identity(m).inv_fn(x) = identity_fn[M](x)
        identity_fn[M](x) = x
        s.contains(linear_equiv_identity(m).inv_fn(x)) = s.contains(x)
        linear_equiv_submodule_map(linear_equiv_identity(m), s).contains(x) = s.contains(x)
    }
}

/// Transporting by the identity equivalence gives the original submodule.
theorem linear_equiv_submodule_map_identity_eq[R: Ring, M: AddCommGroup](
    m: Module[R, M], s: Submodule[R, M]
) {
    s.carrier = m implies linear_equiv_submodule_map(linear_equiv_identity(m), s) = s
} by {
    if s.carrier = m {
        linear_equiv_identity_dst(m)
        linear_equiv_identity(m).dst = m
        linear_map_preimage_submodule_carrier_eq(
            linear_equiv_identity(m).dst, s, linear_equiv_identity(m).inv_fn)
        linear_equiv_submodule_map(linear_equiv_identity(m), s).carrier =
            linear_equiv_identity(m).dst
        linear_equiv_submodule_map(linear_equiv_identity(m), s).carrier = m
        linear_equiv_submodule_map(linear_equiv_identity(m), s).carrier = s.carrier
        forall(x: M) {
            linear_equiv_submodule_map_identity_contains_eq(m, s, x)
            linear_equiv_submodule_map(linear_equiv_identity(m), s).contains(x) = s.contains(x)
        }
        submodule_ext(linear_equiv_submodule_map(linear_equiv_identity(m), s), s)
        linear_equiv_submodule_map(linear_equiv_identity(m), s) = s
    }
}

/// Transporting by the inverse equivalence is preimage along the original forward map.
theorem linear_equiv_submodule_map_symm_contains_eq[R: Ring, M: AddCommGroup, N: AddCommGroup](
    e: LinearEquiv[R, M, N], t: Submodule[R, N], x: M
) {
    t.carrier = e.dst implies
        linear_equiv_submodule_map(linear_equiv_symm(e), t).contains(x) = t.contains(e.to_fn(x))
} by {
    if t.carrier = e.dst {
        linear_equiv_symm_src(e)
        linear_equiv_symm(e).src = e.dst
        linear_equiv_submodule_map_contains_eq(linear_equiv_symm(e), t, x)
        linear_equiv_submodule_map(linear_equiv_symm(e), t).contains(x) =
            t.contains(linear_equiv_symm(e).inv_fn(x))
        linear_equiv_inv_fn_is_linear_map(e)
        let g: M -> N = e.to_fn
        LinearEquiv[R, N, M].new(e.dst, e.src, e.inv_fn, e.to_fn) =
            Option.some(linear_equiv_symm(e))
        LinearEquiv[R, N, M].new(e.dst, e.src, e.inv_fn, g) =
            Option.some(linear_equiv_symm(e))
        g(x) = e.to_fn(x)
        linear_equiv_symm(e).inv_fn(x) = g(x)
        t.contains(linear_equiv_symm(e).inv_fn(x)) = t.contains(e.to_fn(x))
        linear_equiv_submodule_map(linear_equiv_symm(e), t).contains(x) = t.contains(e.to_fn(x))
    }
}

/// Transporting forward and then back by the inverse preserves pointwise membership.
theorem linear_equiv_submodule_map_symm_left_roundtrip_contains_eq[R: Ring, M: AddCommGroup, N: AddCommGroup](
    e: LinearEquiv[R, M, N], s: Submodule[R, M], x: M
) {
    s.carrier = e.src implies
        linear_equiv_submodule_map(linear_equiv_symm(e), linear_equiv_submodule_map(e, s)).contains(x) =
            s.contains(x)
} by {
    if s.carrier = e.src {
        linear_map_preimage_submodule_carrier_eq(e.dst, s, e.inv_fn)
        linear_equiv_submodule_map(e, s).carrier = e.dst
        linear_equiv_submodule_map_symm_contains_eq(e, linear_equiv_submodule_map(e, s), x)
        linear_equiv_submodule_map(linear_equiv_symm(e), linear_equiv_submodule_map(e, s)).contains(x) =
            linear_equiv_submodule_map(e, s).contains(e.to_fn(x))
        linear_equiv_submodule_map_contains_forward(e, s, x)
        linear_equiv_submodule_map(e, s).contains(e.to_fn(x)) = s.contains(x)
        linear_equiv_submodule_map(linear_equiv_symm(e), linear_equiv_submodule_map(e, s)).contains(x) =
            s.contains(x)
    }
}

/// Transporting forward and then back by the inverse gives the original source submodule.
theorem linear_equiv_submodule_map_symm_left_roundtrip_eq[R: Ring, M: AddCommGroup, N: AddCommGroup](
    e: LinearEquiv[R, M, N], s: Submodule[R, M]
) {
    s.carrier = e.src implies
        linear_equiv_submodule_map(linear_equiv_symm(e), linear_equiv_submodule_map(e, s)) = s
} by {
    if s.carrier = e.src {
        linear_equiv_symm_dst(e)
        linear_equiv_symm(e).dst = e.src
        linear_map_preimage_submodule_carrier_eq(e.dst, s, e.inv_fn)
        linear_equiv_submodule_map(e, s).carrier = e.dst
        linear_map_preimage_submodule_carrier_eq(
            linear_equiv_symm(e).dst, linear_equiv_submodule_map(e, s), linear_equiv_symm(e).inv_fn)
        linear_equiv_submodule_map(linear_equiv_symm(e), linear_equiv_submodule_map(e, s)).carrier =
            linear_equiv_symm(e).dst
        linear_equiv_submodule_map(linear_equiv_symm(e), linear_equiv_submodule_map(e, s)).carrier = e.src
        linear_equiv_submodule_map(linear_equiv_symm(e), linear_equiv_submodule_map(e, s)).carrier = s.carrier
        forall(x: M) {
            linear_equiv_submodule_map_symm_left_roundtrip_contains_eq(e, s, x)
            linear_equiv_submodule_map(linear_equiv_symm(e), linear_equiv_submodule_map(e, s)).contains(x) =
                s.contains(x)
        }
        submodule_ext(linear_equiv_submodule_map(linear_equiv_symm(e), linear_equiv_submodule_map(e, s)), s)
        linear_equiv_submodule_map(linear_equiv_symm(e), linear_equiv_submodule_map(e, s)) = s
    }
}

/// Transporting back by the inverse and then forward preserves pointwise membership.
theorem linear_equiv_submodule_map_symm_right_roundtrip_contains_eq[R: Ring, M: AddCommGroup, N: AddCommGroup](
    e: LinearEquiv[R, M, N], t: Submodule[R, N], y: N
) {
    t.carrier = e.dst implies
        linear_equiv_submodule_map(e, linear_equiv_submodule_map(linear_equiv_symm(e), t)).contains(y) =
            t.contains(y)
} by {
    if t.carrier = e.dst {
        linear_equiv_symm_dst(e)
        linear_equiv_symm(e).dst = e.src
        linear_map_preimage_submodule_carrier_eq(
            linear_equiv_symm(e).dst, t, linear_equiv_symm(e).inv_fn)
        linear_equiv_submodule_map(linear_equiv_symm(e), t).carrier = linear_equiv_symm(e).dst
        linear_equiv_submodule_map(linear_equiv_symm(e), t).carrier = e.src
        linear_equiv_submodule_map_contains_eq(e, linear_equiv_submodule_map(linear_equiv_symm(e), t), y)
        linear_equiv_submodule_map(e, linear_equiv_submodule_map(linear_equiv_symm(e), t)).contains(y) =
            linear_equiv_submodule_map(linear_equiv_symm(e), t).contains(e.inv_fn(y))
        linear_equiv_submodule_map_symm_contains_eq(e, t, e.inv_fn(y))
        linear_equiv_submodule_map(linear_equiv_symm(e), t).contains(e.inv_fn(y)) =
            t.contains(e.to_fn(e.inv_fn(y)))
        linear_equiv_right_inverse(e, y)
        e.to_fn(e.inv_fn(y)) = y
        t.contains(e.to_fn(e.inv_fn(y))) = t.contains(y)
        linear_equiv_submodule_map(e, linear_equiv_submodule_map(linear_equiv_symm(e), t)).contains(y) =
            t.contains(y)
    }
}

/// Transporting back by the inverse and then forward gives the original target submodule.
theorem linear_equiv_submodule_map_symm_right_roundtrip_eq[R: Ring, M: AddCommGroup, N: AddCommGroup](
    e: LinearEquiv[R, M, N], t: Submodule[R, N]
) {
    t.carrier = e.dst implies
        linear_equiv_submodule_map(e, linear_equiv_submodule_map(linear_equiv_symm(e), t)) = t
} by {
    if t.carrier = e.dst {
        linear_map_preimage_submodule_carrier_eq(
            linear_equiv_symm(e).dst, t, linear_equiv_symm(e).inv_fn)
        linear_equiv_symm_dst(e)
        linear_equiv_symm(e).dst = e.src
        linear_equiv_submodule_map(linear_equiv_symm(e), t).carrier = e.src
        linear_map_preimage_submodule_carrier_eq(
            e.dst, linear_equiv_submodule_map(linear_equiv_symm(e), t), e.inv_fn)
        linear_equiv_submodule_map(e, linear_equiv_submodule_map(linear_equiv_symm(e), t)).carrier = e.dst
        linear_equiv_submodule_map(e, linear_equiv_submodule_map(linear_equiv_symm(e), t)).carrier = t.carrier
        forall(y: N) {
            linear_equiv_submodule_map_symm_right_roundtrip_contains_eq(e, t, y)
            linear_equiv_submodule_map(e, linear_equiv_submodule_map(linear_equiv_symm(e), t)).contains(y) =
                t.contains(y)
        }
        submodule_ext(linear_equiv_submodule_map(e, linear_equiv_submodule_map(linear_equiv_symm(e), t)), t)
        linear_equiv_submodule_map(e, linear_equiv_submodule_map(linear_equiv_symm(e), t)) = t
    }
}

/// Transporting along a composed equivalence agrees pointwise with iterated transport.
theorem linear_equiv_submodule_map_compose_contains_eq[R: Ring, M: AddCommGroup, N: AddCommGroup, K: AddCommGroup](
    e: LinearEquiv[R, N, K], d: LinearEquiv[R, M, N], h: LinearEquiv[R, M, K],
    s: Submodule[R, M], z: K
) {
    e.src = d.dst and s.carrier = d.src and linear_equiv_compose(e, d) = Option.some(h) implies
        linear_equiv_submodule_map(h, s).contains(z) =
            linear_equiv_submodule_map(e, linear_equiv_submodule_map(d, s)).contains(z)
} by {
    if e.src = d.dst and s.carrier = d.src and linear_equiv_compose(e, d) = Option.some(h) {
        linear_equiv_compose_src(e, d, h)
        h.src = d.src
        linear_equiv_compose_dst(e, d, h)
        h.dst = e.dst
        linear_equiv_submodule_map_contains_eq(h, s, z)
        linear_equiv_submodule_map(h, s).contains(z) = s.contains(h.inv_fn(z))
        let g: K -> M = compose(d.inv_fn, e.inv_fn)
        linear_equiv_compose(e, d) =
            LinearEquiv[R, M, K].new(d.src, e.dst, compose(e.to_fn, d.to_fn), g)
        LinearEquiv[R, M, K].new(d.src, e.dst, compose(e.to_fn, d.to_fn), g) = Option.some(h)
        h.inv_fn = g
        h.inv_fn(z) = g(z)
        g(z) = compose(d.inv_fn, e.inv_fn)(z)
        compose(d.inv_fn, e.inv_fn)(z) = d.inv_fn(e.inv_fn(z))
        s.contains(h.inv_fn(z)) = s.contains(d.inv_fn(e.inv_fn(z)))
        linear_map_preimage_submodule_carrier_eq(d.dst, s, d.inv_fn)
        linear_equiv_submodule_map(d, s).carrier = d.dst
        linear_equiv_submodule_map(d, s).carrier = e.src
        linear_equiv_submodule_map_contains_eq(e, linear_equiv_submodule_map(d, s), z)
        linear_equiv_submodule_map(e, linear_equiv_submodule_map(d, s)).contains(z) =
            linear_equiv_submodule_map(d, s).contains(e.inv_fn(z))
        linear_equiv_submodule_map_contains_eq(d, s, e.inv_fn(z))
        linear_equiv_submodule_map(d, s).contains(e.inv_fn(z)) = s.contains(d.inv_fn(e.inv_fn(z)))
        linear_equiv_submodule_map(e, linear_equiv_submodule_map(d, s)).contains(z) =
            s.contains(d.inv_fn(e.inv_fn(z)))
        linear_equiv_submodule_map(h, s).contains(z) =
            linear_equiv_submodule_map(e, linear_equiv_submodule_map(d, s)).contains(z)
    }
}

/// Transporting along a composed equivalence is iterated transport.
theorem linear_equiv_submodule_map_compose_eq[R: Ring, M: AddCommGroup, N: AddCommGroup, K: AddCommGroup](
    e: LinearEquiv[R, N, K], d: LinearEquiv[R, M, N], h: LinearEquiv[R, M, K],
    s: Submodule[R, M]
) {
    e.src = d.dst and s.carrier = d.src and linear_equiv_compose(e, d) = Option.some(h) implies
        linear_equiv_submodule_map(h, s) =
            linear_equiv_submodule_map(e, linear_equiv_submodule_map(d, s))
} by {
    if e.src = d.dst and s.carrier = d.src and linear_equiv_compose(e, d) = Option.some(h) {
        linear_equiv_compose_dst(e, d, h)
        h.dst = e.dst
        linear_map_preimage_submodule_carrier_eq(h.dst, s, h.inv_fn)
        linear_equiv_submodule_map(h, s).carrier = h.dst
        linear_equiv_submodule_map(h, s).carrier = e.dst
        linear_map_preimage_submodule_carrier_eq(d.dst, s, d.inv_fn)
        linear_equiv_submodule_map(d, s).carrier = d.dst
        linear_equiv_submodule_map(d, s).carrier = e.src
        linear_map_preimage_submodule_carrier_eq(e.dst, linear_equiv_submodule_map(d, s), e.inv_fn)
        linear_equiv_submodule_map(e, linear_equiv_submodule_map(d, s)).carrier = e.dst
        linear_equiv_submodule_map(h, s).carrier =
            linear_equiv_submodule_map(e, linear_equiv_submodule_map(d, s)).carrier
        forall(z: K) {
            linear_equiv_submodule_map_compose_contains_eq(e, d, h, s, z)
            linear_equiv_submodule_map(h, s).contains(z) =
                linear_equiv_submodule_map(e, linear_equiv_submodule_map(d, s)).contains(z)
        }
        submodule_ext(linear_equiv_submodule_map(h, s),
            linear_equiv_submodule_map(e, linear_equiv_submodule_map(d, s)))
        linear_equiv_submodule_map(h, s) =
            linear_equiv_submodule_map(e, linear_equiv_submodule_map(d, s))
    }
}

/// Transport across a bundled linear equivalence reflects and preserves submodule inclusion.
theorem linear_equiv_submodule_map_subset_iff[R: Ring, M: AddCommGroup, N: AddCommGroup](
    e: LinearEquiv[R, M, N], s1: Submodule[R, M], s2: Submodule[R, M]
) {
    s1.carrier = e.src and s2.carrier = e.src implies
        submodule_subset(linear_equiv_submodule_map(e, s1), linear_equiv_submodule_map(e, s2)) =
            submodule_subset(s1, s2)
} by {
    if s1.carrier = e.src and s2.carrier = e.src {
        if submodule_subset(linear_equiv_submodule_map(e, s1), linear_equiv_submodule_map(e, s2)) {
            forall(x: M) {
                if s1.contains(x) {
                    linear_equiv_submodule_map_contains_forward(e, s1, x)
                    linear_equiv_submodule_map(e, s1).contains(e.to_fn(x)) = s1.contains(x)
                    linear_equiv_submodule_map(e, s1).contains(e.to_fn(x))
                    submodule_subset(linear_equiv_submodule_map(e, s1),
                        linear_equiv_submodule_map(e, s2)) = forall(y: N) {
                        linear_equiv_submodule_map(e, s1).contains(y) implies
                            linear_equiv_submodule_map(e, s2).contains(y)
                    }
                    linear_equiv_submodule_map(e, s2).contains(e.to_fn(x))
                    linear_equiv_submodule_map_contains_forward(e, s2, x)
                    linear_equiv_submodule_map(e, s2).contains(e.to_fn(x)) = s2.contains(x)
                    s2.contains(x)
                }
            }
            submodule_subset(s1, s2)
        }
        if submodule_subset(s1, s2) {
            linear_equiv_submodule_map_mono(e, s1, s2)
            submodule_subset(linear_equiv_submodule_map(e, s1), linear_equiv_submodule_map(e, s2))
        }
        submodule_subset(linear_equiv_submodule_map(e, s1), linear_equiv_submodule_map(e, s2)) =
            submodule_subset(s1, s2)
    }
}

/// Transport across a bundled linear equivalence reflects and preserves equality of submodules.
theorem linear_equiv_submodule_map_eq_iff[R: Ring, M: AddCommGroup, N: AddCommGroup](
    e: LinearEquiv[R, M, N], s1: Submodule[R, M], s2: Submodule[R, M]
) {
    s1.carrier = e.src and s2.carrier = e.src implies
        (linear_equiv_submodule_map(e, s1) = linear_equiv_submodule_map(e, s2)) = (s1 = s2)
} by {
    if s1.carrier = e.src and s2.carrier = e.src {
        if linear_equiv_submodule_map(e, s1) = linear_equiv_submodule_map(e, s2) {
            submodule_subset_refl(linear_equiv_submodule_map(e, s1))
            submodule_subset(linear_equiv_submodule_map(e, s1), linear_equiv_submodule_map(e, s2))
            linear_equiv_submodule_map_subset_iff(e, s1, s2)
            submodule_subset(s1, s2)
            submodule_subset_refl(linear_equiv_submodule_map(e, s2))
            submodule_subset(linear_equiv_submodule_map(e, s2), linear_equiv_submodule_map(e, s1))
            linear_equiv_submodule_map_subset_iff(e, s2, s1)
            submodule_subset(s2, s1)
            submodule_subset_antisymm(s1, s2)
            s1 = s2
        }
        if s1 = s2 {
            linear_equiv_submodule_map(e, s1) = linear_equiv_submodule_map(e, s2)
        }
        (linear_equiv_submodule_map(e, s1) = linear_equiv_submodule_map(e, s2)) = (s1 = s2)
    }
}

/// Transport sends the zero submodule to the zero submodule.
theorem linear_equiv_submodule_map_zero_eq_zero[R: Ring, M: AddCommGroup, N: AddCommGroup](
    e: LinearEquiv[R, M, N]
) {
    linear_equiv_submodule_map(e, zero_submodule(e.src)) = zero_submodule(e.dst)
} by {
    zero_submodule_carrier_eq(e.src)
    zero_submodule(e.src).carrier = e.src
    linear_map_preimage_submodule_carrier_eq(e.dst, zero_submodule(e.src), e.inv_fn)
    linear_equiv_submodule_map(e, zero_submodule(e.src)).carrier = e.dst
    zero_submodule_carrier_eq(e.dst)
    linear_equiv_submodule_map(e, zero_submodule(e.src)).carrier = zero_submodule(e.dst).carrier
    forall(y: N) {
        linear_equiv_submodule_map_contains_eq(e, zero_submodule(e.src), y)
        linear_equiv_submodule_map(e, zero_submodule(e.src)).contains(y) =
            zero_submodule(e.src).contains(e.inv_fn(y))
        zero_submodule_contains_eq(e.src, e.inv_fn(y))
        zero_submodule(e.src).contains(e.inv_fn(y)) = (e.inv_fn(y) = M.0)
        linear_equiv_inv_map_eq_zero_iff_eq_zero(e, y)
        (e.inv_fn(y) = M.0) = (y = N.0)
        zero_submodule_contains_eq(e.dst, y)
        zero_submodule(e.dst).contains(y) = (y = N.0)
        linear_equiv_submodule_map(e, zero_submodule(e.src)).contains(y) = zero_submodule(e.dst).contains(y)
    }
    submodule_ext(linear_equiv_submodule_map(e, zero_submodule(e.src)), zero_submodule(e.dst))
    linear_equiv_submodule_map(e, zero_submodule(e.src)) = zero_submodule(e.dst)
}

/// Transport sends the full submodule to the full submodule.
theorem linear_equiv_submodule_map_full_eq_full[R: Ring, M: AddCommGroup, N: AddCommGroup](
    e: LinearEquiv[R, M, N]
) {
    linear_equiv_submodule_map(e, full_submodule(e.src)) = full_submodule(e.dst)
} by {
    full_submodule_carrier_eq(e.src)
    full_submodule(e.src).carrier = e.src
    linear_map_preimage_submodule_carrier_eq(e.dst, full_submodule(e.src), e.inv_fn)
    linear_equiv_submodule_map(e, full_submodule(e.src)).carrier = e.dst
    full_submodule_carrier_eq(e.dst)
    linear_equiv_submodule_map(e, full_submodule(e.src)).carrier = full_submodule(e.dst).carrier
    forall(y: N) {
        linear_equiv_submodule_map_contains_eq(e, full_submodule(e.src), y)
        linear_equiv_submodule_map(e, full_submodule(e.src)).contains(y) =
            full_submodule(e.src).contains(e.inv_fn(y))
        full_submodule_contains_eq(e.src, e.inv_fn(y))
        full_submodule(e.src).contains(e.inv_fn(y)) = true
        full_submodule_contains_eq(e.dst, y)
        full_submodule(e.dst).contains(y) = true
        linear_equiv_submodule_map(e, full_submodule(e.src)).contains(y) = full_submodule(e.dst).contains(y)
    }
    submodule_ext(linear_equiv_submodule_map(e, full_submodule(e.src)), full_submodule(e.dst))
    linear_equiv_submodule_map(e, full_submodule(e.src)) = full_submodule(e.dst)
}

/// Transport preserves intersection of same-source submodules.
theorem linear_equiv_submodule_map_intersection_eq[R: Ring, M: AddCommGroup, N: AddCommGroup](
    e: LinearEquiv[R, M, N], s1: Submodule[R, M], s2: Submodule[R, M]
) {
    s1.carrier = e.src and s2.carrier = e.src implies
        linear_equiv_submodule_map(e, s1.intersection(s2)) =
            linear_equiv_submodule_map(e, s1).intersection(linear_equiv_submodule_map(e, s2))
} by {
    if s1.carrier = e.src and s2.carrier = e.src {
        submodule_intersection_carrier_eq(s1, s2)
        s1.intersection(s2).carrier = s1.carrier
        s1.intersection(s2).carrier = e.src
        linear_map_preimage_submodule_carrier_eq(e.dst, s1.intersection(s2), e.inv_fn)
        linear_equiv_submodule_map(e, s1.intersection(s2)).carrier = e.dst
        linear_map_preimage_submodule_carrier_eq(e.dst, s1, e.inv_fn)
        linear_equiv_submodule_map(e, s1).carrier = e.dst
        linear_map_preimage_submodule_carrier_eq(e.dst, s2, e.inv_fn)
        linear_equiv_submodule_map(e, s2).carrier = e.dst
        submodule_intersection_carrier_eq(linear_equiv_submodule_map(e, s1), linear_equiv_submodule_map(e, s2))
        linear_equiv_submodule_map(e, s1).intersection(linear_equiv_submodule_map(e, s2)).carrier =
            linear_equiv_submodule_map(e, s1).carrier
        linear_equiv_submodule_map(e, s1).intersection(linear_equiv_submodule_map(e, s2)).carrier = e.dst
        linear_equiv_submodule_map(e, s1.intersection(s2)).carrier =
            linear_equiv_submodule_map(e, s1).intersection(linear_equiv_submodule_map(e, s2)).carrier
        forall(y: N) {
            linear_equiv_submodule_map_contains_eq(e, s1.intersection(s2), y)
            linear_equiv_submodule_map(e, s1.intersection(s2)).contains(y) =
                s1.intersection(s2).contains(e.inv_fn(y))
            submodule_intersection_contains_eq_of_carrier_eq(s1, s2, e.inv_fn(y))
            s1.intersection(s2).contains(e.inv_fn(y)) =
                (s1.contains(e.inv_fn(y)) and s2.contains(e.inv_fn(y)))
            linear_equiv_submodule_map_contains_eq(e, s1, y)
            linear_equiv_submodule_map(e, s1).contains(y) = s1.contains(e.inv_fn(y))
            linear_equiv_submodule_map_contains_eq(e, s2, y)
            linear_equiv_submodule_map(e, s2).contains(y) = s2.contains(e.inv_fn(y))
            submodule_intersection_contains_eq_of_carrier_eq(
                linear_equiv_submodule_map(e, s1), linear_equiv_submodule_map(e, s2), y)
            linear_equiv_submodule_map(e, s1).intersection(linear_equiv_submodule_map(e, s2)).contains(y) =
                (linear_equiv_submodule_map(e, s1).contains(y) and
                    linear_equiv_submodule_map(e, s2).contains(y))
            linear_equiv_submodule_map(e, s1).intersection(linear_equiv_submodule_map(e, s2)).contains(y) =
                (s1.contains(e.inv_fn(y)) and s2.contains(e.inv_fn(y)))
            linear_equiv_submodule_map(e, s1.intersection(s2)).contains(y) =
                linear_equiv_submodule_map(e, s1).intersection(linear_equiv_submodule_map(e, s2)).contains(y)
        }
        submodule_ext(linear_equiv_submodule_map(e, s1.intersection(s2)),
            linear_equiv_submodule_map(e, s1).intersection(linear_equiv_submodule_map(e, s2)))
        linear_equiv_submodule_map(e, s1.intersection(s2)) =
            linear_equiv_submodule_map(e, s1).intersection(linear_equiv_submodule_map(e, s2))
    }
}

/// Transport preserves joins of same-source submodules.
theorem linear_equiv_submodule_map_sup_eq[R: Ring, M: AddCommGroup, N: AddCommGroup](
    e: LinearEquiv[R, M, N], s1: Submodule[R, M], s2: Submodule[R, M]
) {
    s1.carrier = e.src and s2.carrier = e.src implies
        linear_equiv_submodule_map(e, submodule_sup(s1, s2)) =
            submodule_sup(linear_equiv_submodule_map(e, s1), linear_equiv_submodule_map(e, s2))
} by {
    if s1.carrier = e.src and s2.carrier = e.src {
        let lhs = linear_equiv_submodule_map(e, submodule_sup(s1, s2))
        let m1 = linear_equiv_submodule_map(e, s1)
        let m2 = linear_equiv_submodule_map(e, s2)
        let rhs = submodule_sup(m1, m2)
        submodule_sup_carrier_eq(s1, s2)
        submodule_sup(s1, s2).carrier = s1.carrier
        submodule_sup(s1, s2).carrier = e.src
        linear_map_preimage_submodule_carrier_eq(e.dst, submodule_sup(s1, s2), e.inv_fn)
        lhs.carrier = e.dst
        linear_map_preimage_submodule_carrier_eq(e.dst, s1, e.inv_fn)
        m1.carrier = e.dst
        linear_map_preimage_submodule_carrier_eq(e.dst, s2, e.inv_fn)
        m2.carrier = e.dst
        submodule_sup_carrier_eq(m1, m2)
        rhs.carrier = m1.carrier
        rhs.carrier = e.dst
        lhs.carrier = rhs.carrier

        submodule_subset_sup_left(s1, s2)
        linear_equiv_submodule_map_mono(e, s1, submodule_sup(s1, s2))
        submodule_subset(m1, lhs)
        submodule_subset_sup_right(s1, s2)
        linear_equiv_submodule_map_mono(e, s2, submodule_sup(s1, s2))
        submodule_subset(m2, lhs)
        submodule_sup_subset_iff(m1, m2, lhs)
        submodule_subset(rhs, lhs)

        linear_equiv_symm_src(e)
        linear_equiv_symm(e).src = e.dst
        linear_equiv_symm_dst(e)
        linear_equiv_symm(e).dst = e.src
        linear_map_preimage_submodule_carrier_eq(linear_equiv_symm(e).dst, rhs, linear_equiv_symm(e).inv_fn)
        linear_equiv_submodule_map(linear_equiv_symm(e), rhs).carrier = linear_equiv_symm(e).dst
        linear_equiv_submodule_map(linear_equiv_symm(e), rhs).carrier = e.src
        forall(x: M) {
            if s1.contains(x) {
                linear_equiv_submodule_map_contains_forward(e, s1, x)
                m1.contains(e.to_fn(x)) = s1.contains(x)
                m1.contains(e.to_fn(x))
                submodule_subset_sup_left(m1, m2)
                submodule_subset(m1, rhs) = forall(y: N) { m1.contains(y) implies rhs.contains(y) }
                rhs.contains(e.to_fn(x))
                linear_equiv_submodule_map_symm_contains_eq(e, rhs, x)
                linear_equiv_submodule_map(linear_equiv_symm(e), rhs).contains(x) = rhs.contains(e.to_fn(x))
                linear_equiv_submodule_map(linear_equiv_symm(e), rhs).contains(x)
            }
        }
        submodule_subset(s1, linear_equiv_submodule_map(linear_equiv_symm(e), rhs))
        forall(x: M) {
            if s2.contains(x) {
                linear_equiv_submodule_map_contains_forward(e, s2, x)
                m2.contains(e.to_fn(x)) = s2.contains(x)
                m2.contains(e.to_fn(x))
                submodule_subset_sup_right(m1, m2)
                submodule_subset(m2, rhs) = forall(y: N) { m2.contains(y) implies rhs.contains(y) }
                rhs.contains(e.to_fn(x))
                linear_equiv_submodule_map_symm_contains_eq(e, rhs, x)
                linear_equiv_submodule_map(linear_equiv_symm(e), rhs).contains(x) = rhs.contains(e.to_fn(x))
                linear_equiv_submodule_map(linear_equiv_symm(e), rhs).contains(x)
            }
        }
        submodule_subset(s2, linear_equiv_submodule_map(linear_equiv_symm(e), rhs))
        submodule_sup_subset_iff(s1, s2, linear_equiv_submodule_map(linear_equiv_symm(e), rhs))
        submodule_subset(submodule_sup(s1, s2), linear_equiv_submodule_map(linear_equiv_symm(e), rhs))
        linear_equiv_submodule_map_mono(e, submodule_sup(s1, s2),
            linear_equiv_submodule_map(linear_equiv_symm(e), rhs))
        submodule_subset(lhs, linear_equiv_submodule_map(e, linear_equiv_submodule_map(linear_equiv_symm(e), rhs)))
        linear_equiv_submodule_map_symm_right_roundtrip_eq(e, rhs)
        linear_equiv_submodule_map(e, linear_equiv_submodule_map(linear_equiv_symm(e), rhs)) = rhs
        submodule_subset(lhs, rhs)

        submodule_subset_antisymm(lhs, rhs)
        lhs = rhs
        linear_equiv_submodule_map(e, submodule_sup(s1, s2)) =
            submodule_sup(linear_equiv_submodule_map(e, s1), linear_equiv_submodule_map(e, s2))
    }
}
