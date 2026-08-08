from real import Real, lt_mul_pos_left, lte_mul_nonneg_left
from algebra.module.module import Module, module_smul_zero_left, module_smul_zero_right, module_smul_neg_right, module_smul_one, module_smul_add_right
from algebra.module.vector_space import vector_space_smul_inverse_left, vector_space_smul_inverse_right
from data.basic.set import Set, set_image, maps_into_set_image, set_image_contains_witness, subset_contains_eq
from algebra.module.normed_add_comm_group import NormedAddCommGroup, norm_non_negative, norm_distance, norm_distance_left_eq,
    norm_open_ball, norm_closed_ball
from algebra.module.normed_field import NormedField
numerals Real

/// True if scalar multiplication is compatible with the norm:
/// `‖r · v‖ = ‖r‖ · ‖v‖`.
define norm_smul_constraint[K: NormedField, V: NormedAddCommGroup](smul: K -> V -> V) -> Bool {
    forall(r: K, v: V) {
        smul(r, v).norm = r.norm * v.norm
    }
}

/// A normed vector space over a normed field `K`: a `Module[K, V]` whose scalar
/// action is compatible with the norm on `V`.
structure NormedSpace[K: NormedField, V: NormedAddCommGroup] {
    /// The underlying `K`-module structure.
    module: Module[K, V]
} constraint {
    norm_smul_constraint(module.smul)
}

/// The defining norm-smul compatibility: `‖r · v‖ = ‖r‖ · ‖v‖`.
theorem normed_space_norm_smul[K: NormedField, V: NormedAddCommGroup](
    ns: NormedSpace[K, V], r: K, v: V) {
    ns.module.smul(r, v).norm = r.norm * v.norm
} by {
    norm_smul_constraint[K, V](ns.module.smul) = forall(a: K, x: V) {
        ns.module.smul(a, x).norm = a.norm * x.norm
    }
}

/// Scaling by zero produces the zero vector.
theorem normed_space_smul_zero_scalar[K: NormedField, V: NormedAddCommGroup](
    ns: NormedSpace[K, V], v: V) {
    ns.module.smul(K.0, v) = V.0
} by {
    module_smul_zero_left(ns.module, v)
}

/// Scaling the zero vector produces the zero vector.
theorem normed_space_smul_zero_vector[K: NormedField, V: NormedAddCommGroup](
    ns: NormedSpace[K, V], r: K) {
    ns.module.smul(r, V.0) = V.0
} by {
    module_smul_zero_right(ns.module, r)
}

/// Negating the scalar preserves the norm of a scaled vector.
theorem normed_space_norm_smul_neg_scalar[K: NormedField, V: NormedAddCommGroup](
    ns: NormedSpace[K, V], r: K, v: V) {
    ns.module.smul(-r, v).norm = ns.module.smul(r, v).norm
} by {
    normed_space_norm_smul(ns, -r, v)
    normed_space_norm_smul(ns, r, v)
}

/// Negating the vector preserves the norm of a scaled vector.
theorem normed_space_norm_smul_neg_vector[K: NormedField, V: NormedAddCommGroup](
    ns: NormedSpace[K, V], r: K, v: V) {
    ns.module.smul(r, -v).norm = ns.module.smul(r, v).norm
} by {
    module_smul_neg_right(ns.module, r, v)
}

/// Scaling by a unit-norm scalar preserves the norm.
theorem normed_space_norm_smul_unit[K: NormedField, V: NormedAddCommGroup](
    ns: NormedSpace[K, V], r: K, v: V) {
    r.norm = 1 implies ns.module.smul(r, v).norm = v.norm
} by {
    if r.norm = 1 {
        normed_space_norm_smul(ns, r, v)
        1 * v.norm = v.norm
    }
}

/// Scaling by the field unit acts as the identity.
theorem normed_space_smul_one[K: NormedField, V: NormedAddCommGroup](
    ns: NormedSpace[K, V], v: V) {
    ns.module.smul(K.1, v) = v
} by {
    module_smul_one(ns.module, v)
}

/// The norm of a scaled vector is non-negative.
theorem normed_space_norm_smul_non_negative[K: NormedField, V: NormedAddCommGroup](
    ns: NormedSpace[K, V], r: K, v: V) {
    0 <= ns.module.smul(r, v).norm
} by {
    norm_non_negative(ns.module.smul(r, v))
}

/// Scaled triangle inequality: the norm of a scaled sum is bounded by the
/// sum of scaled norms.
theorem normed_space_norm_smul_add[K: NormedField, V: NormedAddCommGroup](
    ns: NormedSpace[K, V], r: K, u: V, v: V) {
    ns.module.smul(r, u + v).norm <= r.norm * u.norm + r.norm * v.norm
} by {
    module_smul_add_right(ns.module, r, u, v)
    (ns.module.smul(r, u) + ns.module.smul(r, v)).norm <= ns.module.smul(r, u).norm + ns.module.smul(r, v).norm
    normed_space_norm_smul(ns, r, u)
    normed_space_norm_smul(ns, r, v)
}

/// The induced distance scales linearly under scalar multiplication.
theorem normed_space_norm_distance_smul[K: NormedField, V: NormedAddCommGroup](
    ns: NormedSpace[K, V], r: K, u: V, v: V) {
    norm_distance(ns.module.smul(r, u), ns.module.smul(r, v)) = r.norm * norm_distance(u, v)
} by {
    module_smul_neg_right(ns.module, r, v)
    module_smul_add_right(ns.module, r, u, -v)
    (ns.module.smul(r, u) - ns.module.smul(r, v)).norm = ns.module.smul(r, u - v).norm
    normed_space_norm_smul(ns, r, u - v)
    norm_distance_left_eq(ns.module.smul(r, u), ns.module.smul(r, v))
    norm_distance_left_eq(u, v)
}

/// Scalar multiplication by a fixed scalar, as a named map.
define normed_space_smul_map[K: NormedField, V: NormedAddCommGroup](
    ns: NormedSpace[K, V], r: K, v: V) -> V {
    ns.module.smul(r, v)
}

/// Image of a set under scalar multiplication by a fixed scalar.
define normed_space_smul_set[K: NormedField, V: NormedAddCommGroup](
    ns: NormedSpace[K, V], r: K, s: Set[V]) -> Set[V] {
    set_image(s, normed_space_smul_map(ns, r))
}

/// Scaling an element of a set gives an element of the scaled set.
theorem normed_space_smul_set_contains_smul[K: NormedField, V: NormedAddCommGroup](
    ns: NormedSpace[K, V], r: K, s: Set[V], v: V) {
    s.contains(v) implies normed_space_smul_set(ns, r, s).contains(ns.module.smul(r, v))
} by {
    if s.contains(v) {
        maps_into_set_image(s, normed_space_smul_map(ns, r), v)
        normed_space_smul_set(ns, r, s).contains(ns.module.smul(r, v))
    }
}

/// Membership in a scalar-multiplied set by a nonzero scalar is membership
/// after inverse scalar multiplication.
theorem normed_space_smul_set_contains_iff_of_ne_zero[K: NormedField, V: NormedAddCommGroup](
    ns: NormedSpace[K, V], r: K, s: Set[V], y: V
) {
    r != K.0 implies
        normed_space_smul_set(ns, r, s).contains(y) = s.contains(ns.module.smul(r.inverse, y))
} by {
    if r != K.0 {
        if normed_space_smul_set(ns, r, s).contains(y) {
            normed_space_smul_set(ns, r, s) = set_image(s, normed_space_smul_map(ns, r))
            set_image(s, normed_space_smul_map(ns, r)).contains(y)
            set_image_contains_witness(s, normed_space_smul_map(ns, r), y)
            let x: V satisfy {
                s.contains(x) and y = normed_space_smul_map(ns, r, x)
            }
            normed_space_smul_map(ns, r, x) = ns.module.smul(r, x)
            y = ns.module.smul(r, x)
            ns.module.smul(r.inverse, y) = ns.module.smul(r.inverse, ns.module.smul(r, x))
            vector_space_smul_inverse_left(ns.module, r, x)
            ns.module.smul(r.inverse, ns.module.smul(r, x)) = x
            ns.module.smul(r.inverse, y) = x
            s.contains(ns.module.smul(r.inverse, y))
        }
        if s.contains(ns.module.smul(r.inverse, y)) {
            maps_into_set_image(s, normed_space_smul_map(ns, r), ns.module.smul(r.inverse, y))
            set_image(s, normed_space_smul_map(ns, r)).contains(
                normed_space_smul_map(ns, r, ns.module.smul(r.inverse, y)))
            normed_space_smul_map(ns, r, ns.module.smul(r.inverse, y)) =
                ns.module.smul(r, ns.module.smul(r.inverse, y))
            vector_space_smul_inverse_right(ns.module, r, y)
            ns.module.smul(r, ns.module.smul(r.inverse, y)) = y
            set_image(s, normed_space_smul_map(ns, r)).contains(y)
            normed_space_smul_set(ns, r, s) = set_image(s, normed_space_smul_map(ns, r))
            normed_space_smul_set(ns, r, s).contains(y)
        }
        normed_space_smul_set(ns, r, s).contains(y) = s.contains(ns.module.smul(r.inverse, y))
    }
}

/// Scaling an open ball by a scalar of positive norm lands in the open ball
/// whose center and radius are scaled accordingly.
theorem normed_space_smul_open_ball_subset[K: NormedField, V: NormedAddCommGroup](
    ns: NormedSpace[K, V], r: K, x: V, eps: Real) {
    r.norm.is_positive implies
        normed_space_smul_set(ns, r, norm_open_ball(x, eps)).subset(
            norm_open_ball(ns.module.smul(r, x), r.norm * eps))
} by {
    if r.norm.is_positive {
        let s = normed_space_smul_set(ns, r, norm_open_ball(x, eps))
        let b = norm_open_ball(ns.module.smul(r, x), r.norm * eps)
        forall(y: V) {
            if s.contains(y) {
                set_image_contains_witness(norm_open_ball(x, eps), normed_space_smul_map(ns, r), y)
                let w: V satisfy {
                    norm_open_ball(x, eps).contains(w) and y = normed_space_smul_map(ns, r, w)
                }
                normed_space_norm_distance_smul(ns, r, w, x)
                lt_mul_pos_left(norm_distance(w, x), eps, r.norm)
                r.norm * norm_distance(w, x) < r.norm * eps
                norm_distance(y, ns.module.smul(r, x)) < r.norm * eps
                b.contains(y)
            }
        }
        subset_contains_eq(s, b)
        s.subset(b)
        normed_space_smul_set(ns, r, norm_open_ball(x, eps)).subset(
            norm_open_ball(ns.module.smul(r, x), r.norm * eps))
    }
}

/// Scaling a closed ball lands in the closed ball whose center and radius are
/// scaled by the scalar norm.
theorem normed_space_smul_closed_ball_subset[K: NormedField, V: NormedAddCommGroup](
    ns: NormedSpace[K, V], r: K, x: V, eps: Real) {
    normed_space_smul_set(ns, r, norm_closed_ball(x, eps)).subset(
        norm_closed_ball(ns.module.smul(r, x), r.norm * eps))
} by {
    let s = normed_space_smul_set(ns, r, norm_closed_ball(x, eps))
    let b = norm_closed_ball(ns.module.smul(r, x), r.norm * eps)
    forall(y: V) {
        if s.contains(y) {
            set_image_contains_witness(norm_closed_ball(x, eps), normed_space_smul_map(ns, r), y)
            let w: V satisfy {
                norm_closed_ball(x, eps).contains(w) and y = normed_space_smul_map(ns, r, w)
            }
            norm_non_negative(r)
            not r.norm.is_negative
            normed_space_norm_distance_smul(ns, r, w, x)
            lte_mul_nonneg_left(norm_distance(w, x), eps, r.norm)
            r.norm * norm_distance(w, x) <= r.norm * eps
            norm_distance(y, ns.module.smul(r, x)) <= r.norm * eps
            b.contains(y)
        }
    }
    subset_contains_eq(s, b)
    s.subset(b)
}
