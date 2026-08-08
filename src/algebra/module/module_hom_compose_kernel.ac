from algebra.ring.ring import Ring
from algebra.add_comm_group import AddCommGroup
from data.basic.functions import is_injective_fn
from algebra.module.module_hom import ModuleHom, module_hom_zero, module_hom_compose,
    module_hom_compose_src, module_hom_compose_hom_at
from algebra.module.submodule import module_hom_kernel, module_hom_kernel_carrier,
    module_hom_kernel_map_eq_zero, module_hom_kernel_contains_of_map_eq_zero,
    submodule_subset, submodule_subset_antisymm, submodule_ext,
    module_hom_kernel_implies_zero_of_injective
from algebra.module.module_hom_submodule_map_comap import module_hom_submodule_comap,
    module_hom_submodule_comap_carrier_eq, module_hom_submodule_comap_contains_eq

/// An element of the kernel of the inner homomorphism lies in the kernel of a bundled composition.
theorem module_hom_compose_kernel_contains_of_kernel[R: Ring, M: AddCommGroup, N: AddCommGroup, K: AddCommGroup](
    f: ModuleHom[R, N, K], g: ModuleHom[R, M, N], h: ModuleHom[R, M, K], x: M
) {
    module_hom_compose(f, g) = Option.some(h) and module_hom_kernel(g).contains(x)
        implies module_hom_kernel(h).contains(x)
} by {
    if module_hom_compose(f, g) = Option.some(h) and module_hom_kernel(g).contains(x) {
        module_hom_kernel_map_eq_zero(g, x)
        module_hom_zero(f)
        f.hom(g.hom(x)) = K.0
        module_hom_compose_hom_at(f, g, h, x)
        module_hom_kernel_contains_of_map_eq_zero(h, x)
        module_hom_kernel(h).contains(x)
    }
}

/// The kernel submodule of the inner homomorphism is contained in the kernel submodule of a bundled composition.
theorem module_hom_kernel_inner_subset_module_hom_compose_kernel[R: Ring, M: AddCommGroup, N: AddCommGroup, K: AddCommGroup](
    f: ModuleHom[R, N, K], g: ModuleHom[R, M, N], h: ModuleHom[R, M, K]
) {
    module_hom_compose(f, g) = Option.some(h)
        implies submodule_subset(module_hom_kernel(g), module_hom_kernel(h))
} by {
    if module_hom_compose(f, g) = Option.some(h) {
        forall(x: M) {
            if module_hom_kernel(g).contains(x) {
                module_hom_compose_kernel_contains_of_kernel(f, g, h, x)
                module_hom_kernel(h).contains(x)
            }
        }
    }
}

/// If the outer homomorphism is injective, an element of the kernel of a bundled composition lies in the kernel of the inner homomorphism.
theorem module_hom_compose_kernel_contains_implies_kernel_inner_of_injective_outer[R: Ring, M: AddCommGroup, N: AddCommGroup, K: AddCommGroup](
    f: ModuleHom[R, N, K], g: ModuleHom[R, M, N], h: ModuleHom[R, M, K], x: M
) {
    module_hom_compose(f, g) = Option.some(h) and is_injective_fn(f.hom)
        and module_hom_kernel(h).contains(x)
        implies module_hom_kernel(g).contains(x)
} by {
    if module_hom_compose(f, g) = Option.some(h) and is_injective_fn(f.hom)
        and module_hom_kernel(h).contains(x) {
        module_hom_kernel_map_eq_zero(h, x)
        module_hom_compose_hom_at(f, g, h, x)
        module_hom_kernel_contains_of_map_eq_zero(f, g.hom(x))
        module_hom_kernel(f).contains(g.hom(x))
        module_hom_kernel_implies_zero_of_injective(f, g.hom(x))
        module_hom_kernel_contains_of_map_eq_zero(g, x)
        module_hom_kernel(g).contains(x)
    }
}

/// If the outer homomorphism is injective, the kernel submodule of a bundled composition is contained in the kernel submodule of the inner homomorphism.
theorem module_hom_compose_kernel_subset_kernel_inner_of_injective_outer[R: Ring, M: AddCommGroup, N: AddCommGroup, K: AddCommGroup](
    f: ModuleHom[R, N, K], g: ModuleHom[R, M, N], h: ModuleHom[R, M, K]
) {
    module_hom_compose(f, g) = Option.some(h) and is_injective_fn(f.hom)
        implies submodule_subset(module_hom_kernel(h), module_hom_kernel(g))
} by {
    if module_hom_compose(f, g) = Option.some(h) and is_injective_fn(f.hom) {
        forall(x: M) {
            if module_hom_kernel(h).contains(x) {
                module_hom_compose_kernel_contains_implies_kernel_inner_of_injective_outer(f, g, h, x)
                module_hom_kernel(g).contains(x)
            }
        }
    }
}

/// If the outer homomorphism is injective, a bundled composition has exactly the inner kernel.
theorem module_hom_compose_kernel_eq_kernel_inner_of_injective_outer[R: Ring, M: AddCommGroup, N: AddCommGroup, K: AddCommGroup](
    f: ModuleHom[R, N, K], g: ModuleHom[R, M, N], h: ModuleHom[R, M, K]
) {
    module_hom_compose(f, g) = Option.some(h) and is_injective_fn(f.hom)
        implies module_hom_kernel(h) = module_hom_kernel(g)
} by {
    if module_hom_compose(f, g) = Option.some(h) and is_injective_fn(f.hom) {
        let kernel_h = module_hom_kernel(h)
        let kernel_g = module_hom_kernel(g)
        module_hom_compose_kernel_subset_kernel_inner_of_injective_outer(f, g, h)
        module_hom_kernel_inner_subset_module_hom_compose_kernel(f, g, h)
        submodule_subset(kernel_g, kernel_h)
        module_hom_kernel_carrier(h)
        module_hom_compose_src(f, g, h)
        module_hom_kernel_carrier(g)
        kernel_h.carrier = kernel_g.carrier
        submodule_subset_antisymm(kernel_h, kernel_g)
        kernel_h = kernel_g
        module_hom_kernel(h) = module_hom_kernel(g)
    }
}

/// The kernel of a bundled composition is the comap of the outer kernel along the inner homomorphism.
theorem module_hom_compose_kernel_eq_comap_kernel[R: Ring, M: AddCommGroup, N: AddCommGroup, K: AddCommGroup](
    f: ModuleHom[R, N, K], g: ModuleHom[R, M, N], h: ModuleHom[R, M, K]
) {
    f.src = g.dst and module_hom_compose(f, g) = Option.some(h) implies
        module_hom_kernel(h) = module_hom_submodule_comap(g, module_hom_kernel(f))
} by {
    if f.src = g.dst and module_hom_compose(f, g) = Option.some(h) {
        let lhs = module_hom_kernel(h)
        let rhs = module_hom_submodule_comap(g, module_hom_kernel(f))
        module_hom_kernel_carrier(h)
        lhs.carrier = h.src
        module_hom_compose_src(f, g, h)
        h.src = g.src
        lhs.carrier = g.src
        module_hom_submodule_comap_carrier_eq(g, module_hom_kernel(f))
        rhs.carrier = g.src
        lhs.carrier = rhs.carrier
        module_hom_kernel_carrier(f)
        module_hom_kernel(f).carrier = f.src
        module_hom_kernel(f).carrier = g.dst
        forall(x: M) {
            module_hom_submodule_comap_contains_eq(g, module_hom_kernel(f), x)
            rhs.contains(x) = module_hom_kernel(f).contains(g.hom(x))
            if lhs.contains(x) {
                module_hom_kernel_map_eq_zero(h, x)
                h.hom(x) = K.0
                module_hom_compose_hom_at(f, g, h, x)
                h.hom(x) = f.hom(g.hom(x))
                f.hom(g.hom(x)) = K.0
                module_hom_kernel_contains_of_map_eq_zero(f, g.hom(x))
                module_hom_kernel(f).contains(g.hom(x))
                rhs.contains(x)
            }
            if rhs.contains(x) {
                module_hom_kernel(f).contains(g.hom(x))
                module_hom_kernel_map_eq_zero(f, g.hom(x))
                f.hom(g.hom(x)) = K.0
                module_hom_compose_hom_at(f, g, h, x)
                h.hom(x) = f.hom(g.hom(x))
                h.hom(x) = K.0
                module_hom_kernel_contains_of_map_eq_zero(h, x)
                lhs.contains(x)
            }
            lhs.contains(x) = rhs.contains(x)
        }
        submodule_ext(lhs, rhs)
        lhs = rhs
        module_hom_kernel(h) = module_hom_submodule_comap(g, module_hom_kernel(f))
    }
}
