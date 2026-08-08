/// Submodule image and the map-comap Galois bridge for linear maps.

from algebra.ring.ring import Ring
from algebra.add_comm_group import AddCommGroup
from algebra.module.module import Module
from algebra.module.module_hom import is_linear_map, identity_fn_is_linear_map
from data.basic.functions import identity_fn
from data.basic.relation_transport import predicate_subset, predicate_subset_step
from data.basic.set import Set, set_image, maps_into_set_image, set_image_contains_witness
from algebra.module.submodule import Submodule, submodule_subset, submodule_subset_antisymm,
    submodule_as_set_contains_eq, set_subset_submodule, submodule_closure,
    submodule_closure_carrier_eq, submodule_closure_contains_of_set_contains,
    submodule_closure_subset_of_set_subset, linear_map_preimage_submodule,
    linear_map_preimage_submodule_contains_of_image_in,
    linear_map_preimage_submodule_image_in_of_contains

/// Submodule inclusion transports membership of a single element.
theorem submodule_subset_contains_at[R: Ring, M: AddCommGroup](
    a: Submodule[R, M], b: Submodule[R, M], x: M
) {
    submodule_subset(a, b) and a.contains(x) implies b.contains(x)
} by {
    if submodule_subset(a, b) and a.contains(x) {
        submodule_subset(a, b) = predicate_subset(a.contains, b.contains)
        predicate_subset(a.contains, b.contains)
        predicate_subset_step(a.contains, b.contains, x)
        b.contains(x)
    }
}

/// A pointwise membership implication gives predicate inclusion.
theorem predicate_subset_intro[T](p: T -> Bool, q: T -> Bool) {
    (forall(x: T) { p(x) implies q(x) }) implies predicate_subset(p, q)
} by {
    if forall(x: T) { p(x) implies q(x) } {
        forall(y: T) {
            if p(y) {
                q(y)
            }
        }
    }
}

/// A pointwise membership implication gives submodule inclusion.
theorem submodule_subset_intro[R: Ring, M: AddCommGroup](
    a: Submodule[R, M], b: Submodule[R, M]
) {
    (forall(x: M) { a.contains(x) implies b.contains(x) }) implies submodule_subset(a, b)
} by {
    if forall(x: M) { a.contains(x) implies b.contains(x) } {
        predicate_subset_intro(a.contains, b.contains)
        predicate_subset(a.contains, b.contains)
        submodule_subset(a, b) = predicate_subset(a.contains, b.contains)
        submodule_subset(a, b)
    }
}

/// A pointwise membership implication gives containment of a set in a submodule.
theorem set_subset_submodule_intro[R: Ring, M: AddCommGroup](a: Set[M], s: Submodule[R, M]) {
    (forall(x: M) { a.contains(x) implies s.contains(x) }) implies set_subset_submodule(a, s)
} by {
    if forall(x: M) { a.contains(x) implies s.contains(x) } {
        set_subset_submodule(a, s) = forall(y: M) {
            a.contains(y) implies s.contains(y)
        }
        set_subset_submodule(a, s)
    }
}

/// True if `y` is the image under `f` of some member of the source submodule `s`.
define linear_map_submodule_image_witness[R: Ring, M: AddCommGroup, N: AddCommGroup](
    s: Submodule[R, M], f: M -> N, y: N
) -> Bool {
    exists(x: M) {
        s.contains(x) and f(x) = y
    }
}

/// The submodule image of a source submodule under a map is the submodule generated by
/// the pointwise image of its underlying set.
define linear_map_submodule_image[R: Ring, M: AddCommGroup, N: AddCommGroup](
    src: Module[R, M], dst: Module[R, N], f: M -> N, s: Submodule[R, M]
) -> Submodule[R, N] {
    submodule_closure(dst, set_image(s.as_set, f))
}

/// The submodule image is the closure of the pointwise image of the underlying set.
theorem linear_map_submodule_image_eq_closure[R: Ring, M: AddCommGroup, N: AddCommGroup](
    src: Module[R, M], dst: Module[R, N], f: M -> N, s: Submodule[R, M]
) {
    linear_map_submodule_image(src, dst, f, s) = submodule_closure(dst, set_image(s.as_set, f))
} by {
    linear_map_submodule_image(src, dst, f, s) = submodule_closure(dst, set_image(s.as_set, f))
}

/// The submodule image has the declared target as its ambient module.
theorem linear_map_submodule_image_carrier_eq[R: Ring, M: AddCommGroup, N: AddCommGroup](
    src: Module[R, M], dst: Module[R, N], f: M -> N, s: Submodule[R, M]
) {
    linear_map_submodule_image(src, dst, f, s).carrier = dst
} by {
    submodule_closure_carrier_eq(dst, set_image(s.as_set, f))
}

/// A witness for pointwise image membership gives membership in the submodule image.
theorem linear_map_submodule_image_contains_of_witness[R: Ring, M: AddCommGroup, N: AddCommGroup](
    src: Module[R, M], dst: Module[R, N], f: M -> N, s: Submodule[R, M], y: N
) {
    linear_map_submodule_image_witness(s, f, y) implies
        linear_map_submodule_image(src, dst, f, s).contains(y)
} by {
    if linear_map_submodule_image_witness(s, f, y) {
        let x: M satisfy {
            s.contains(x) and f(x) = y
        }
        submodule_as_set_contains_eq(s, x)
        s.as_set.contains(x)
        maps_into_set_image(s.as_set, f, x)
        set_image(s.as_set, f).contains(f(x))
        set_image(s.as_set, f).contains(y)
        submodule_closure_contains_of_set_contains(dst, set_image(s.as_set, f), y)
        submodule_closure(dst, set_image(s.as_set, f)).contains(y)
        linear_map_submodule_image(src, dst, f, s).contains(y)
    }
}

/// A source-submodule element maps into the submodule image.
theorem linear_map_submodule_image_contains_apply[R: Ring, M: AddCommGroup, N: AddCommGroup](
    src: Module[R, M], dst: Module[R, N], f: M -> N, s: Submodule[R, M], x: M
) {
    s.contains(x) implies linear_map_submodule_image(src, dst, f, s).contains(f(x))
} by {
    if s.contains(x) {
        exists(t: M) {
            s.contains(t) and f(t) = f(x)
        }
        linear_map_submodule_image_witness(s, f, f(x))
        linear_map_submodule_image_contains_of_witness(src, dst, f, s, f(x))
        linear_map_submodule_image(src, dst, f, s).contains(f(x))
    }
}

/// If the source submodule is contained in the target preimage, then the submodule image is
/// contained in the target submodule.
theorem linear_map_submodule_image_subset_of_preimage_subset[R: Ring, M: AddCommGroup, N: AddCommGroup](
    src: Module[R, M], dst: Module[R, N], f: M -> N, s: Submodule[R, M], t: Submodule[R, N]
) {
    is_linear_map(src, dst, f) and s.carrier = src and t.carrier = dst
        and submodule_subset(s, linear_map_preimage_submodule(src, t, f))
    implies submodule_subset(linear_map_submodule_image(src, dst, f, s), t)
} by {
    if is_linear_map(src, dst, f) and s.carrier = src and t.carrier = dst
        and submodule_subset(s, linear_map_preimage_submodule(src, t, f)) {
        forall(y: N) {
            if set_image(s.as_set, f).contains(y) {
                set_image_contains_witness(s.as_set, f, y)
                let x: M satisfy {
                    s.as_set.contains(x) and y = f(x)
                }
                submodule_as_set_contains_eq(s, x)
                s.contains(x)
                submodule_subset_contains_at(s, linear_map_preimage_submodule(src, t, f), x)
                linear_map_preimage_submodule(src, t, f).contains(x)
                is_linear_map(src, t.carrier, f)
                linear_map_preimage_submodule_image_in_of_contains(src, t, f, x)
                t.contains(f(x))
                t.contains(y)
            }
        }
        set_subset_submodule_intro(set_image(s.as_set, f), t)
        set_subset_submodule(set_image(s.as_set, f), t)
        submodule_closure_subset_of_set_subset(dst, set_image(s.as_set, f), t)
        submodule_subset(submodule_closure(dst, set_image(s.as_set, f)), t)
        submodule_subset(linear_map_submodule_image(src, dst, f, s), t)
    }
}

/// If the submodule image is contained in the target submodule, then the source submodule is
/// contained in the target preimage.
theorem linear_map_submodule_image_subset_preimage_of_image_subset[R: Ring, M: AddCommGroup, N: AddCommGroup](
    src: Module[R, M], dst: Module[R, N], f: M -> N, s: Submodule[R, M], t: Submodule[R, N]
) {
    is_linear_map(src, dst, f) and s.carrier = src and t.carrier = dst
        and submodule_subset(linear_map_submodule_image(src, dst, f, s), t)
    implies submodule_subset(s, linear_map_preimage_submodule(src, t, f))
} by {
    if is_linear_map(src, dst, f) and s.carrier = src and t.carrier = dst
        and submodule_subset(linear_map_submodule_image(src, dst, f, s), t) {
        forall(x: M) {
            if s.contains(x) {
                linear_map_submodule_image_contains_apply(src, dst, f, s, x)
                linear_map_submodule_image(src, dst, f, s).contains(f(x))
                submodule_subset_contains_at(linear_map_submodule_image(src, dst, f, s), t, f(x))
                t.contains(f(x))
                is_linear_map(src, t.carrier, f)
                linear_map_preimage_submodule_contains_of_image_in(src, t, f, x)
                linear_map_preimage_submodule(src, t, f).contains(x)
            }
        }
        submodule_subset_intro(s, linear_map_preimage_submodule(src, t, f))
        submodule_subset(s, linear_map_preimage_submodule(src, t, f))
    }
}

/// The submodule image is contained in a target submodule exactly when the source submodule
/// is contained in the target preimage.
theorem linear_map_submodule_image_subset_iff_preimage[R: Ring, M: AddCommGroup, N: AddCommGroup](
    src: Module[R, M], dst: Module[R, N], f: M -> N, s: Submodule[R, M], t: Submodule[R, N]
) {
    is_linear_map(src, dst, f) and s.carrier = src and t.carrier = dst implies
        submodule_subset(linear_map_submodule_image(src, dst, f, s), t) =
            submodule_subset(s, linear_map_preimage_submodule(src, t, f))
} by {
    if is_linear_map(src, dst, f) and s.carrier = src and t.carrier = dst {
        if submodule_subset(linear_map_submodule_image(src, dst, f, s), t) {
            linear_map_submodule_image_subset_preimage_of_image_subset(src, dst, f, s, t)
            submodule_subset(s, linear_map_preimage_submodule(src, t, f))
        }
        if submodule_subset(s, linear_map_preimage_submodule(src, t, f)) {
            linear_map_submodule_image_subset_of_preimage_subset(src, dst, f, s, t)
            submodule_subset(linear_map_submodule_image(src, dst, f, s), t)
        }
        submodule_subset(linear_map_submodule_image(src, dst, f, s), t) =
            submodule_subset(s, linear_map_preimage_submodule(src, t, f))
    }
}

/// A member of a submodule belongs to its identity-map image.
theorem linear_map_submodule_image_identity_contains_of_contains[R: Ring, M: AddCommGroup](
    src: Module[R, M], s: Submodule[R, M], x: M
) {
    s.contains(x) implies linear_map_submodule_image(src, src, identity_fn[M], s).contains(x)
} by {
    if s.contains(x) {
        linear_map_submodule_image_contains_apply(src, src, identity_fn[M], s, x)
        linear_map_submodule_image(src, src, identity_fn[M], s).contains(identity_fn[M](x))
        identity_fn[M](x) = x
        linear_map_submodule_image(src, src, identity_fn[M], s).contains(x)
    }
}

/// The image of a submodule under the identity linear map is the original submodule.
theorem linear_map_submodule_image_identity[R: Ring, M: AddCommGroup](
    src: Module[R, M], s: Submodule[R, M]
) {
    s.carrier = src implies linear_map_submodule_image(src, src, identity_fn[M], s) = s
} by {
    let image = linear_map_submodule_image(src, src, identity_fn[M], s)
    if s.carrier = src {
        identity_fn_is_linear_map(src)
        linear_map_submodule_image_carrier_eq(src, src, identity_fn[M], s)
        image.carrier = src
        image.carrier = s.carrier
        forall(y: M) {
            if set_image(s.as_set, identity_fn[M]).contains(y) {
                set_image_contains_witness(s.as_set, identity_fn[M], y)
                let x: M satisfy {
                    s.as_set.contains(x) and y = identity_fn[M](x)
                }
                submodule_as_set_contains_eq(s, x)
                s.contains(x)
                identity_fn[M](x) = x
                y = x
                s.contains(y)
            }
        }
        set_subset_submodule_intro(set_image(s.as_set, identity_fn[M]), s)
        set_subset_submodule(set_image(s.as_set, identity_fn[M]), s)
        submodule_closure_subset_of_set_subset(src, set_image(s.as_set, identity_fn[M]), s)
        submodule_subset(image, s)
        forall(x: M) {
            if s.contains(x) {
                linear_map_submodule_image_identity_contains_of_contains(src, s, x)
                image.contains(x)
            }
        }
        submodule_subset(s, image)
        submodule_subset_antisymm(image, s)
        image = s
    }
}
