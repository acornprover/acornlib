/// Finite support for functions valued in algebraic structures with zero.

from finite_set import FiniteSet, fs_union
from data.basic.logic import exists_intro
from pair import Pair, pair_ext
from data.basic.set import not_contains_of_subset_not_contains, not_union_contains_eq
from algebra.zero import Zero
from algebra.add_monoid import AddMonoid
from algebra.add_group import AddGroup
from semiring import Semiring
from data.basic.function_algebra import pointwise_add, pointwise_zero, pointwise_neg, pointwise_mul
from algebra.product_algebra import pair_add, pair_mul, pair_zero, pair_neg, pair_add_first,
    pair_add_second, pair_mul_first, pair_mul_second, pair_neg_first, pair_neg_second

/// True if a function is zero outside a specified finite set.
define has_support_in[T, A: Zero](f: T -> A, s: FiniteSet[T]) -> Bool {
    forall(t: T) {
        not s.contains(t) implies f(t) = A.0
    }
}

/// True if a function is zero outside some finite set.
define is_finitely_supported[T, A: Zero](f: T -> A) -> Bool {
    exists(s: FiniteSet[T]) {
        has_support_in(f, s)
    }
}

/// A displayed finite support is a finite-support witness.
theorem is_finitely_supported_of_has_support_in[T, A: Zero](f: T -> A, s: FiniteSet[T]) {
    has_support_in(f, s) implies is_finitely_supported(f)
} by {
    if has_support_in(f, s) {
        exists(u: FiniteSet[T]) {
            u = s and has_support_in(f, u)
        }
    }
}

/// A finite-support witness gives zero outside the witnessing set.
theorem has_support_in_apply_zero[T, A: Zero](f: T -> A, s: FiniteSet[T], t: T) {
    has_support_in(f, s) and not s.contains(t) implies f(t) = A.0
} by {
    if has_support_in(f, s) and not s.contains(t) {
        has_support_in(f, s) = forall(x: T) {
            not s.contains(x) implies f(x) = A.0
        }
        f(t) = A.0
    }
}

/// Enlarging a finite support set preserves the support witness.
theorem has_support_in_subset[T, A: Zero](f: T -> A, s: FiniteSet[T], t: FiniteSet[T]) {
    has_support_in(f, s) and s.underlying_set.subset(t.underlying_set) implies has_support_in(f, t)
} by {
    if has_support_in(f, s) and s.underlying_set.subset(t.underlying_set) {
        forall(x: T) {
            if not t.contains(x) {
                not_contains_of_subset_not_contains(s.underlying_set, t.underlying_set, x)
                not s.contains(x)
                has_support_in_apply_zero[T, A](f, s, x)
                f(x) = A.0
            }
        }
    }
}

/// The zero function is zero outside any finite set.
theorem pointwise_zero_has_support_in[T, A: Zero](s: FiniteSet[T]) {
    has_support_in(pointwise_zero[T, A], s)
} by {
    forall(t: T) {
        if not s.contains(t) {
            pointwise_zero[T, A](t) = A.0
        }
    }
}

/// The zero function is finitely supported.
theorem pointwise_zero_is_finitely_supported[T, A: Zero] {
    is_finitely_supported(pointwise_zero[T, A])
} by {
    pointwise_zero_has_support_in[T, A](FiniteSet[T].empty)
}

/// The pointwise sum of functions supported in two finite sets is supported in their union.
theorem pointwise_add_has_support_in_union[T, A: AddMonoid](
    f: T -> A,
    g: T -> A,
    s: FiniteSet[T],
    u: FiniteSet[T]
) {
    has_support_in(f, s) and has_support_in(g, u) implies
    has_support_in(pointwise_add(f, g), fs_union(s, u))
} by {
    if has_support_in(f, s) and has_support_in(g, u) {
        forall(t: T) {
            if not fs_union(s, u).contains(t) {
                fs_union(s, u).underlying_set = s.underlying_set.union(u.underlying_set)
                fs_union(s, u).contains(t) = s.underlying_set.union(u.underlying_set).contains(t)
                not s.underlying_set.union(u.underlying_set).contains(t)
                not_union_contains_eq[T](s.underlying_set, u.underlying_set, t)
                not s.underlying_set.contains(t) and not u.underlying_set.contains(t)
                not s.contains(t)
                not u.contains(t)
                has_support_in_apply_zero[T, A](f, s, t)
                has_support_in_apply_zero[T, A](g, u, t)
                f(t) = A.0
                g(t) = A.0
                pointwise_add(f, g, t) = f(t) + g(t)
                pointwise_add(f, g, t) = A.0 + A.0
                A.0 + A.0 = A.0
                pointwise_add(f, g, t) = A.0
            }
        }
    }
}

/// The pointwise sum of finitely supported functions is finitely supported.
theorem pointwise_add_is_finitely_supported[T, A: AddMonoid](f: T -> A, g: T -> A) {
    is_finitely_supported(f) and is_finitely_supported(g) implies
    is_finitely_supported(pointwise_add(f, g))
} by {
    if is_finitely_supported(f) and is_finitely_supported(g) {
        let s: FiniteSet[T] satisfy {
            has_support_in(f, s)
        }
        let u: FiniteSet[T] satisfy {
            has_support_in(g, u)
        }
        pointwise_add_has_support_in_union(f, g, s, u)
        has_support_in(pointwise_add(f, g), fs_union(s, u))
        exists(v: FiniteSet[T]) {
            v = fs_union(s, u) and has_support_in(pointwise_add(f, g), v)
        }
        is_finitely_supported(pointwise_add(f, g))
    }
}

/// The pointwise negation of a function supported in a finite set is supported in that set.
theorem pointwise_neg_has_support_in[T, A: AddGroup](f: T -> A, s: FiniteSet[T]) {
    has_support_in(f, s) implies has_support_in(pointwise_neg(f), s)
} by {
    if has_support_in(f, s) {
        forall(t: T) {
            if not s.contains(t) {
                has_support_in_apply_zero[T, A](f, s, t)
                f(t) = A.0
                pointwise_neg(f, t) = -f(t)
                pointwise_neg(f, t) = -A.0
                A.0 + -A.0 = A.0
                A.0 + A.0 = A.0
                -A.0 = A.0
                pointwise_neg(f, t) = A.0
            }
        }
    }
}

/// The pointwise negation of a finitely supported function is finitely supported.
theorem pointwise_neg_is_finitely_supported[T, A: AddGroup](f: T -> A) {
    is_finitely_supported(f) implies is_finitely_supported(pointwise_neg(f))
} by {
    if is_finitely_supported(f) {
        let s: FiniteSet[T] satisfy {
            has_support_in(f, s)
        }
        pointwise_neg_has_support_in[T, A](f, s)
        exists(u: FiniteSet[T]) {
            u = s and has_support_in(pointwise_neg(f), u)
        }
    }
}

/// Multiplying on the left by a function supported in a finite set is supported in that set.
theorem pointwise_mul_has_support_in_left[T, A: Semiring](f: T -> A, g: T -> A, s: FiniteSet[T]) {
    has_support_in(f, s) implies has_support_in(pointwise_mul(f, g), s)
} by {
    if has_support_in(f, s) {
        forall(t: T) {
            if not s.contains(t) {
                has_support_in_apply_zero[T, A](f, s, t)
                f(t) = A.0
                pointwise_mul(f, g, t) = f(t) * g(t)
                pointwise_mul(f, g, t) = A.0 * g(t)
                A.0 * g(t) = A.0
                pointwise_mul(f, g, t) = A.0
            }
        }
    }
}

/// Multiplying on the right by a function supported in a finite set is supported in that set.
theorem pointwise_mul_has_support_in_right[T, A: Semiring](f: T -> A, g: T -> A, s: FiniteSet[T]) {
    has_support_in(g, s) implies has_support_in(pointwise_mul(f, g), s)
} by {
    if has_support_in(g, s) {
        forall(t: T) {
            if not s.contains(t) {
                has_support_in_apply_zero[T, A](g, s, t)
                g(t) = A.0
                pointwise_mul(f, g, t) = f(t) * g(t)
                pointwise_mul(f, g, t) = f(t) * A.0
                f(t) * A.0 = A.0
                pointwise_mul(f, g, t) = A.0
            }
        }
    }
}

/// A product with a finitely supported left factor is finitely supported.
theorem pointwise_mul_is_finitely_supported_left[T, A: Semiring](f: T -> A, g: T -> A) {
    is_finitely_supported(f) implies is_finitely_supported(pointwise_mul(f, g))
} by {
    if is_finitely_supported(f) {
        let s: FiniteSet[T] satisfy {
            has_support_in(f, s)
        }
        pointwise_mul_has_support_in_left[T, A](f, g, s)
        exists(u: FiniteSet[T]) {
            u = s and has_support_in(pointwise_mul(f, g), u)
        }
    }
}

/// A product with a finitely supported right factor is finitely supported.
theorem pointwise_mul_is_finitely_supported_right[T, A: Semiring](f: T -> A, g: T -> A) {
    is_finitely_supported(g) implies is_finitely_supported(pointwise_mul(f, g))
} by {
    if is_finitely_supported(g) {
        let s: FiniteSet[T] satisfy {
            has_support_in(g, s)
        }
        pointwise_mul_has_support_in_right[T, A](f, g, s)
        exists(u: FiniteSet[T]) {
            u = s and has_support_in(pointwise_mul(f, g), u)
        }
    }
}

/// The pointwise product of finitely supported functions is finitely supported.
theorem pointwise_mul_is_finitely_supported[T, A: Semiring](f: T -> A, g: T -> A) {
    is_finitely_supported(f) and is_finitely_supported(g) implies
    is_finitely_supported(pointwise_mul(f, g))
} by {
    if is_finitely_supported(f) and is_finitely_supported(g) {
        pointwise_mul_is_finitely_supported_left[T, A](f, g)
    }
}

/// The first coordinate of a product-valued function.
define pair_function_first[T, A, B](f: T -> Pair[A, B], t: T) -> A {
    f(t).first
}

/// The second coordinate of a product-valued function.
define pair_function_second[T, A, B](f: T -> Pair[A, B], t: T) -> B {
    f(t).second
}

/// The pointwise zero for product-valued functions.
define pointwise_pair_zero[T, A: Zero, B: Zero](t: T) -> Pair[A, B] {
    pair_zero[A, B]
}

/// The pointwise sum of product-valued functions.
define pointwise_pair_add[T, A: AddMonoid, B: AddMonoid](
    f: T -> Pair[A, B],
    g: T -> Pair[A, B],
    t: T
) -> Pair[A, B] {
    pair_add(f(t), g(t))
}

/// The pointwise negation of a product-valued function.
define pointwise_pair_neg[T, A: AddGroup, B: AddGroup](f: T -> Pair[A, B], t: T) -> Pair[A, B] {
    pair_neg(f(t))
}

/// The pointwise product of product-valued functions.
define pointwise_pair_mul[T, A: Semiring, B: Semiring](
    f: T -> Pair[A, B],
    g: T -> Pair[A, B],
    t: T
) -> Pair[A, B] {
    pair_mul(f(t), g(t))
}

/// True if a product-valued function is zero outside a specified finite set.
define has_pair_support_in[T, A: Zero, B: Zero](f: T -> Pair[A, B], s: FiniteSet[T]) -> Bool {
    forall(t: T) {
        not s.contains(t) implies f(t) = pair_zero[A, B]
    }
}

/// True if a product-valued function is zero outside some finite set.
define is_pair_finitely_supported[T, A: Zero, B: Zero](f: T -> Pair[A, B]) -> Bool {
    exists(s: FiniteSet[T]) {
        has_pair_support_in(f, s)
    }
}

/// A displayed product-valued finite support is a finite-support witness.
theorem is_pair_finitely_supported_of_has_pair_support_in[T, A: Zero, B: Zero](
    f: T -> Pair[A, B],
    s: FiniteSet[T]
) {
    has_pair_support_in(f, s) implies is_pair_finitely_supported(f)
} by {
    if has_pair_support_in(f, s) {
        exists(u: FiniteSet[T]) {
            u = s and has_pair_support_in(f, u)
        }
    }
}

/// A product finite-support witness gives the product zero outside the witnessing set.
theorem has_pair_support_in_apply_zero[T, A: Zero, B: Zero](f: T -> Pair[A, B], s: FiniteSet[T], t: T) {
    has_pair_support_in(f, s) and not s.contains(t) implies f(t) = pair_zero[A, B]
} by {
    if has_pair_support_in(f, s) and not s.contains(t) {
        has_pair_support_in(f, s) = forall(x: T) {
            not s.contains(x) implies f(x) = pair_zero[A, B]
        }
        f(t) = pair_zero[A, B]
    }
}

/// Product support gives support of the first coordinate.
theorem has_pair_support_in_first[T, A: Zero, B: Zero](f: T -> Pair[A, B], s: FiniteSet[T]) {
    has_pair_support_in(f, s) implies has_support_in(pair_function_first(f), s)
} by {
    if has_pair_support_in(f, s) {
        forall(t: T) {
            if not s.contains(t) {
                has_pair_support_in_apply_zero(f, s, t)
                f(t) = pair_zero[A, B]
                pair_function_first(f, t) = f(t).first
                pair_zero[A, B].first = A.0
                pair_function_first(f, t) = A.0
            }
        }
    }
}

/// Product support gives support of the second coordinate.
theorem has_pair_support_in_second[T, A: Zero, B: Zero](f: T -> Pair[A, B], s: FiniteSet[T]) {
    has_pair_support_in(f, s) implies has_support_in(pair_function_second(f), s)
} by {
    if has_pair_support_in(f, s) {
        forall(t: T) {
            if not s.contains(t) {
                has_pair_support_in_apply_zero(f, s, t)
                f(t) = pair_zero[A, B]
                pair_function_second(f, t) = f(t).second
                pair_zero[A, B].second = B.0
                pair_function_second(f, t) = B.0
            }
        }
    }
}

/// Coordinate support in the same finite set gives product support.
theorem has_pair_support_in_of_coordinate_support[T, A: Zero, B: Zero](
    f: T -> Pair[A, B],
    s: FiniteSet[T]
) {
    has_support_in(pair_function_first(f), s) and has_support_in(pair_function_second(f), s)
    implies has_pair_support_in(f, s)
} by {
    if has_support_in(pair_function_first(f), s) and has_support_in(pair_function_second(f), s) {
        forall(t: T) {
            if not s.contains(t) {
                has_support_in_apply_zero[T, A](pair_function_first(f), s, t)
                pair_function_first(f, t) = A.0
                has_support_in_apply_zero[T, B](pair_function_second(f), s, t)
                pair_function_second(f, t) = B.0
                let value = f(t)
                value.first = pair_function_first(f, t)
                value.first = A.0
                value.second = pair_function_second(f, t)
                value.second = B.0
                pair_ext(value, pair_zero[A, B])
                f(t) = pair_zero[A, B]
            }
        }
    }
}

/// Coordinate support in finite sets gives product support in their union.
theorem has_pair_support_in_union_of_coordinate_support[T, A: Zero, B: Zero](
    f: T -> Pair[A, B],
    s: FiniteSet[T],
    u: FiniteSet[T]
) {
    has_support_in(pair_function_first(f), s) and has_support_in(pair_function_second(f), u)
    implies has_pair_support_in(f, fs_union(s, u))
} by {
    if has_support_in(pair_function_first(f), s) and has_support_in(pair_function_second(f), u) {
        forall(t: T) {
            if not fs_union(s, u).contains(t) {
                fs_union(s, u).underlying_set = s.underlying_set.union(u.underlying_set)
                fs_union(s, u).contains(t) = s.underlying_set.union(u.underlying_set).contains(t)
                not s.underlying_set.union(u.underlying_set).contains(t)
                not_union_contains_eq[T](s.underlying_set, u.underlying_set, t)
                not s.underlying_set.contains(t) and not u.underlying_set.contains(t)
                not s.contains(t)
                not u.contains(t)
                has_support_in_apply_zero[T, A](pair_function_first(f), s, t)
                pair_function_first(f, t) = A.0
                has_support_in_apply_zero[T, B](pair_function_second(f), u, t)
                pair_function_second(f, t) = B.0
                let value = f(t)
                value.first = pair_function_first(f, t)
                value.first = A.0
                value.second = pair_function_second(f, t)
                value.second = B.0
                pair_ext(value, pair_zero[A, B])
                f(t) = pair_zero[A, B]
            }
        }
    }
}

/// Product finite support gives finite support of the first coordinate.
theorem pair_finitely_supported_first[T, A: Zero, B: Zero](f: T -> Pair[A, B]) {
    is_pair_finitely_supported(f) implies is_finitely_supported(pair_function_first(f))
} by {
    if is_pair_finitely_supported(f) {
        let s: FiniteSet[T] satisfy {
            has_pair_support_in(f, s)
        }
        has_pair_support_in_first(f, s)
        exists(u: FiniteSet[T]) {
            u = s and has_support_in(pair_function_first(f), u)
        }
    }
}

/// Product finite support gives finite support of the second coordinate.
theorem pair_finitely_supported_second[T, A: Zero, B: Zero](f: T -> Pair[A, B]) {
    is_pair_finitely_supported(f) implies is_finitely_supported(pair_function_second(f))
} by {
    if is_pair_finitely_supported(f) {
        let s: FiniteSet[T] satisfy {
            has_pair_support_in(f, s)
        }
        has_pair_support_in_second(f, s)
        exists(u: FiniteSet[T]) {
            u = s and has_support_in(pair_function_second(f), u)
        }
    }
}

/// Coordinate supports in finite sets imply finite support of the product-valued function.
theorem pair_finitely_supported_of_coordinate_supports[T, A: Zero, B: Zero](
    f: T -> Pair[A, B],
    s: FiniteSet[T],
    u: FiniteSet[T]
) {
    has_support_in(pair_function_first(f), s) and has_support_in(pair_function_second(f), u)
    implies is_pair_finitely_supported(f)
} by {
    if has_support_in(pair_function_first(f), s) and has_support_in(pair_function_second(f), u) {
        has_pair_support_in_union_of_coordinate_support(f, s, u)
        has_pair_support_in(f, fs_union(s, u))
        exists_intro(function(v: FiniteSet[T]) {
            has_pair_support_in(f, v)
        }, fs_union(s, u))
        exists(v: FiniteSet[T]) {
            has_pair_support_in(f, v)
        }
        is_pair_finitely_supported(f)
    }
}

/// A product-valued function is finitely supported when both coordinate functions are finitely supported.
theorem pair_finitely_supported_of_coordinates[T, A: Zero, B: Zero](f: T -> Pair[A, B]) {
    is_finitely_supported(pair_function_first(f)) and is_finitely_supported(pair_function_second(f))
    implies is_pair_finitely_supported(f)
} by {
    if is_finitely_supported(pair_function_first(f)) and is_finitely_supported(pair_function_second(f)) {
        let s: FiniteSet[T] satisfy {
            has_support_in(pair_function_first(f), s)
        }
        let u: FiniteSet[T] satisfy {
            has_support_in(pair_function_second(f), u)
        }
        has_support_in(pair_function_first(f), s) and has_support_in(pair_function_second(f), u)
        has_pair_support_in_union_of_coordinate_support(f, s, u)
        has_pair_support_in(f, fs_union(s, u))
        exists_intro(function(v: FiniteSet[T]) {
            has_pair_support_in(f, v)
        }, fs_union(s, u))
        exists(v: FiniteSet[T]) {
            has_pair_support_in(f, v)
        }
        is_pair_finitely_supported(f)
    }
}

/// Product-valued finite support is equivalent to finite support of both coordinates.
theorem pair_finitely_supported_iff_coordinates[T, A: Zero, B: Zero](f: T -> Pair[A, B]) {
    is_pair_finitely_supported(f) =
        (is_finitely_supported(pair_function_first(f)) and is_finitely_supported(pair_function_second(f)))
} by {
    if is_pair_finitely_supported(f) {
        pair_finitely_supported_first(f)
        pair_finitely_supported_second(f)
        is_finitely_supported(pair_function_first(f)) and is_finitely_supported(pair_function_second(f))
    }
    if is_finitely_supported(pair_function_first(f)) and is_finitely_supported(pair_function_second(f)) {
        pair_finitely_supported_of_coordinates(f)
        is_pair_finitely_supported(f)
    }
}

/// The product-valued zero function is zero outside any finite set.
theorem pointwise_pair_zero_has_support_in[T, A: Zero, B: Zero](s: FiniteSet[T]) {
    has_pair_support_in(pointwise_pair_zero[T, A, B], s)
} by {
    forall(t: T) {
        if not s.contains(t) {
            pointwise_pair_zero[T, A, B](t) = pair_zero[A, B]
        }
    }
}

/// The product-valued zero function is finitely supported.
theorem pointwise_pair_zero_is_finitely_supported[T, A: Zero, B: Zero] {
    is_pair_finitely_supported(pointwise_pair_zero[T, A, B])
} by {
    pointwise_pair_zero_has_support_in[T, A, B](FiniteSet[T].empty)
    exists(s: FiniteSet[T]) {
        s = FiniteSet[T].empty and has_pair_support_in(pointwise_pair_zero[T, A, B], s)
    }
}

/// The pointwise sum of product-valued functions supported in two finite sets is supported in their union.
theorem pointwise_pair_add_has_support_in_union[T, A: AddMonoid, B: AddMonoid](
    f: T -> Pair[A, B],
    g: T -> Pair[A, B],
    s: FiniteSet[T],
    u: FiniteSet[T]
) {
    has_pair_support_in(f, s) and has_pair_support_in(g, u) implies
    has_pair_support_in(pointwise_pair_add(f, g), fs_union(s, u))
} by {
    if has_pair_support_in(f, s) and has_pair_support_in(g, u) {
        forall(t: T) {
            if not fs_union(s, u).contains(t) {
                fs_union(s, u).underlying_set = s.underlying_set.union(u.underlying_set)
                fs_union(s, u).contains(t) = s.underlying_set.union(u.underlying_set).contains(t)
                not s.underlying_set.union(u.underlying_set).contains(t)
                not_union_contains_eq[T](s.underlying_set, u.underlying_set, t)
                not s.underlying_set.contains(t) and not u.underlying_set.contains(t)
                not s.contains(t)
                not u.contains(t)
                has_pair_support_in_apply_zero(f, s, t)
                f(t) = pair_zero[A, B]
                has_pair_support_in_apply_zero(g, u, t)
                g(t) = pair_zero[A, B]
                let value = pointwise_pair_add(f, g, t)
                value = pair_add(f(t), g(t))
                pair_add_first(f(t), g(t))
                value.first = f(t).first + g(t).first
                f(t).first = pair_zero[A, B].first
                f(t).first = A.0
                g(t).first = pair_zero[A, B].first
                g(t).first = A.0
                value.first = A.0
                pair_add_second(f(t), g(t))
                value.second = f(t).second + g(t).second
                f(t).second = pair_zero[A, B].second
                f(t).second = B.0
                g(t).second = pair_zero[A, B].second
                g(t).second = B.0
                value.second = B.0
                pair_ext(value, pair_zero[A, B])
                pointwise_pair_add(f, g, t) = pair_zero[A, B]
            }
        }
    }
}

/// Supports in finite sets imply finite support of the pointwise product-valued sum.
theorem pointwise_pair_add_is_finitely_supported_of_supports[T, A: AddMonoid, B: AddMonoid](
    f: T -> Pair[A, B],
    g: T -> Pair[A, B],
    s: FiniteSet[T],
    u: FiniteSet[T]
) {
    has_pair_support_in(f, s) and has_pair_support_in(g, u) implies
    is_pair_finitely_supported(pointwise_pair_add(f, g))
} by {
    if has_pair_support_in(f, s) and has_pair_support_in(g, u) {
        pointwise_pair_add_has_support_in_union(f, g, s, u)
        has_pair_support_in(pointwise_pair_add(f, g), fs_union(s, u))
        exists_intro(function(v: FiniteSet[T]) {
            has_pair_support_in(pointwise_pair_add(f, g), v)
        }, fs_union(s, u))
        exists(v: FiniteSet[T]) {
            has_pair_support_in(pointwise_pair_add(f, g), v)
        }
        is_pair_finitely_supported(pointwise_pair_add(f, g))
    }
}

/// The pointwise sum of finitely supported product-valued functions is finitely supported.
theorem pointwise_pair_add_is_finitely_supported[T, A: AddMonoid, B: AddMonoid](
    f: T -> Pair[A, B],
    g: T -> Pair[A, B]
) {
    is_pair_finitely_supported(f) and is_pair_finitely_supported(g) implies
    is_pair_finitely_supported(pointwise_pair_add(f, g))
} by {
    if is_pair_finitely_supported(f) and is_pair_finitely_supported(g) {
        let s: FiniteSet[T] satisfy {
            has_pair_support_in(f, s)
        }
        let u: FiniteSet[T] satisfy {
            has_pair_support_in(g, u)
        }
        has_pair_support_in(f, s) and has_pair_support_in(g, u)
        pointwise_pair_add_has_support_in_union(f, g, s, u)
        has_pair_support_in(pointwise_pair_add(f, g), fs_union(s, u))
        exists_intro(function(v: FiniteSet[T]) {
            has_pair_support_in(pointwise_pair_add(f, g), v)
        }, fs_union(s, u))
        exists(v: FiniteSet[T]) {
            has_pair_support_in(pointwise_pair_add(f, g), v)
        }
        is_pair_finitely_supported(pointwise_pair_add(f, g))
    }
}

/// The pointwise negation of a supported product-valued function is supported in the same finite set.
theorem pointwise_pair_neg_has_support_in[T, A: AddGroup, B: AddGroup](
    f: T -> Pair[A, B],
    s: FiniteSet[T]
) {
    has_pair_support_in(f, s) implies has_pair_support_in(pointwise_pair_neg(f), s)
} by {
    if has_pair_support_in(f, s) {
        forall(t: T) {
            if not s.contains(t) {
                has_pair_support_in_apply_zero(f, s, t)
                f(t) = pair_zero[A, B]
                let value = pointwise_pair_neg(f, t)
                value = pair_neg(f(t))
                pair_neg_first(f(t))
                value.first = -f(t).first
                f(t).first = pair_zero[A, B].first
                f(t).first = A.0
                A.0 + -A.0 = A.0
                A.0 + A.0 = A.0
                -A.0 = A.0
                value.first = A.0
                pair_neg_second(f(t))
                value.second = -f(t).second
                f(t).second = pair_zero[A, B].second
                f(t).second = B.0
                B.0 + -B.0 = B.0
                B.0 + B.0 = B.0
                -B.0 = B.0
                value.second = B.0
                pair_ext(value, pair_zero[A, B])
                pointwise_pair_neg(f, t) = pair_zero[A, B]
            }
        }
    }
}

/// Multiplying on the left by a supported product-valued function is supported in the same finite set.
theorem pointwise_pair_mul_has_support_in_left[T, A: Semiring, B: Semiring](
    f: T -> Pair[A, B],
    g: T -> Pair[A, B],
    s: FiniteSet[T]
) {
    has_pair_support_in(f, s) implies has_pair_support_in(pointwise_pair_mul(f, g), s)
} by {
    if has_pair_support_in(f, s) {
        forall(t: T) {
            if not s.contains(t) {
                has_pair_support_in_apply_zero(f, s, t)
                f(t) = pair_zero[A, B]
                let value = pointwise_pair_mul(f, g, t)
                value = pair_mul(f(t), g(t))
                pair_mul_first(f(t), g(t))
                value.first = f(t).first * g(t).first
                f(t).first = pair_zero[A, B].first
                f(t).first = A.0
                value.first = A.0
                pair_mul_second(f(t), g(t))
                value.second = f(t).second * g(t).second
                f(t).second = pair_zero[A, B].second
                f(t).second = B.0
                value.second = B.0
                pair_ext(value, pair_zero[A, B])
                pointwise_pair_mul(f, g, t) = pair_zero[A, B]
            }
        }
    }
}

/// Multiplying on the right by a supported product-valued function is supported in the same finite set.
theorem pointwise_pair_mul_has_support_in_right[T, A: Semiring, B: Semiring](
    f: T -> Pair[A, B],
    g: T -> Pair[A, B],
    s: FiniteSet[T]
) {
    has_pair_support_in(g, s) implies has_pair_support_in(pointwise_pair_mul(f, g), s)
} by {
    if has_pair_support_in(g, s) {
        forall(t: T) {
            if not s.contains(t) {
                has_pair_support_in_apply_zero(g, s, t)
                g(t) = pair_zero[A, B]
                let value = pointwise_pair_mul(f, g, t)
                value = pair_mul(f(t), g(t))
                pair_mul_first(f(t), g(t))
                value.first = f(t).first * g(t).first
                g(t).first = pair_zero[A, B].first
                g(t).first = A.0
                value.first = A.0
                pair_mul_second(f(t), g(t))
                value.second = f(t).second * g(t).second
                g(t).second = pair_zero[A, B].second
                g(t).second = B.0
                value.second = B.0
                pair_ext(value, pair_zero[A, B])
                pointwise_pair_mul(f, g, t) = pair_zero[A, B]
            }
        }
    }
}

/// The product-valued negation of a finitely supported function is finitely supported.
theorem pointwise_pair_neg_is_finitely_supported[T, A: AddGroup, B: AddGroup](f: T -> Pair[A, B]) {
    is_pair_finitely_supported(f) implies is_pair_finitely_supported(pointwise_pair_neg(f))
} by {
    if is_pair_finitely_supported(f) {
        let s: FiniteSet[T] satisfy {
            has_pair_support_in(f, s)
        }
        pointwise_pair_neg_has_support_in(f, s)
        exists(u: FiniteSet[T]) {
            u = s and has_pair_support_in(pointwise_pair_neg(f), u)
        }
    }
}

/// A product-valued product with a finitely supported left factor is finitely supported.
theorem pointwise_pair_mul_is_finitely_supported_left[T, A: Semiring, B: Semiring](
    f: T -> Pair[A, B],
    g: T -> Pair[A, B]
) {
    is_pair_finitely_supported(f) implies is_pair_finitely_supported(pointwise_pair_mul(f, g))
} by {
    if is_pair_finitely_supported(f) {
        let s: FiniteSet[T] satisfy {
            has_pair_support_in(f, s)
        }
        pointwise_pair_mul_has_support_in_left(f, g, s)
        exists(u: FiniteSet[T]) {
            u = s and has_pair_support_in(pointwise_pair_mul(f, g), u)
        }
    }
}

/// A product-valued product with a finitely supported right factor is finitely supported.
theorem pointwise_pair_mul_is_finitely_supported_right[T, A: Semiring, B: Semiring](
    f: T -> Pair[A, B],
    g: T -> Pair[A, B]
) {
    is_pair_finitely_supported(g) implies is_pair_finitely_supported(pointwise_pair_mul(f, g))
} by {
    if is_pair_finitely_supported(g) {
        let s: FiniteSet[T] satisfy {
            has_pair_support_in(g, s)
        }
        pointwise_pair_mul_has_support_in_right(f, g, s)
        exists(u: FiniteSet[T]) {
            u = s and has_pair_support_in(pointwise_pair_mul(f, g), u)
        }
    }
}

/// The product-valued product of finitely supported functions is finitely supported.
theorem pointwise_pair_mul_is_finitely_supported[T, A: Semiring, B: Semiring](
    f: T -> Pair[A, B],
    g: T -> Pair[A, B]
) {
    is_pair_finitely_supported(f) and is_pair_finitely_supported(g) implies
    is_pair_finitely_supported(pointwise_pair_mul(f, g))
} by {
    if is_pair_finitely_supported(f) and is_pair_finitely_supported(g) {
        pointwise_pair_mul_is_finitely_supported_left(f, g)
    }
}
