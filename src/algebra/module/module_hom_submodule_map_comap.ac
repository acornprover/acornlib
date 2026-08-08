/// Bundled module-hom submodule map/comap API and transport facts.

from algebra.ring.ring import Ring
from algebra.add_comm_group import AddCommGroup
from algebra.module.module_hom import ModuleHom, is_linear_map, module_hom_is_linear_map
from algebra.module.submodule import Submodule, submodule_subset, submodule_subset_antisymm,
    zero_submodule, full_submodule, full_submodule_carrier_eq,
    full_submodule_contains_everything, linear_map_kernel_submodule,
    linear_map_preimage_submodule, linear_map_preimage_submodule_carrier_eq,
    linear_map_preimage_submodule_contains_eq,
    linear_map_preimage_submodule_contains_of_image_in,
    linear_map_preimage_submodule_image_in_of_contains,
    linear_map_preimage_submodule_mono,
    linear_map_preimage_submodule_full_subset,
    full_submodule_subset_linear_map_preimage_full,
    linear_map_preimage_submodule_zero_subset_kernel,
    linear_map_kernel_subset_preimage_submodule_zero,
    module_hom_kernel, module_hom_image, module_hom_kernel_carrier,
    module_hom_image_carrier, module_hom_image_contains_value, module_hom_image_elim
from algebra.module.submodule_map_comap import linear_map_submodule_image,
    linear_map_submodule_image_carrier_eq, linear_map_submodule_image_contains_apply,
    linear_map_submodule_image_subset_iff_preimage, submodule_subset_intro

/// The image (map) of a submodule under a bundled module homomorphism.
define module_hom_submodule_map[R: Ring, M: AddCommGroup, N: AddCommGroup](
    f: ModuleHom[R, M, N], s: Submodule[R, M]
) -> Submodule[R, N] {
    linear_map_submodule_image(f.src, f.dst, f.hom, s)
}

/// The preimage (comap) of a submodule under a bundled module homomorphism.
define module_hom_submodule_comap[R: Ring, M: AddCommGroup, N: AddCommGroup](
    f: ModuleHom[R, M, N], t: Submodule[R, N]
) -> Submodule[R, M] {
    linear_map_preimage_submodule(f.src, t, f.hom)
}

/// A mapped submodule has the destination module as ambient module.
theorem module_hom_submodule_map_carrier_eq[R: Ring, M: AddCommGroup, N: AddCommGroup](
    f: ModuleHom[R, M, N], s: Submodule[R, M]
) {
    module_hom_submodule_map(f, s).carrier = f.dst
} by {
    linear_map_submodule_image_carrier_eq(f.src, f.dst, f.hom, s)
}

/// A comapped submodule has the source module as ambient module.
theorem module_hom_submodule_comap_carrier_eq[R: Ring, M: AddCommGroup, N: AddCommGroup](
    f: ModuleHom[R, M, N], t: Submodule[R, N]
) {
    module_hom_submodule_comap(f, t).carrier = f.src
} by {
    linear_map_preimage_submodule_carrier_eq(f.src, t, f.hom)
}

/// A member of the source submodule maps into the mapped submodule.
theorem module_hom_submodule_map_contains_apply[R: Ring, M: AddCommGroup, N: AddCommGroup](
    f: ModuleHom[R, M, N], s: Submodule[R, M], x: M
) {
    s.contains(x) implies module_hom_submodule_map(f, s).contains(f.hom(x))
} by {
    if s.contains(x) {
        linear_map_submodule_image_contains_apply(f.src, f.dst, f.hom, s, x)
        module_hom_submodule_map(f, s).contains(f.hom(x))
    }
}

/// Membership in a comap is membership of the image in the target submodule.
theorem module_hom_submodule_comap_contains_eq[R: Ring, M: AddCommGroup, N: AddCommGroup](
    f: ModuleHom[R, M, N], t: Submodule[R, N], x: M
) {
    t.carrier = f.dst implies module_hom_submodule_comap(f, t).contains(x) = t.contains(f.hom(x))
} by {
    if t.carrier = f.dst {
        module_hom_is_linear_map(f)
        is_linear_map(f.src, f.dst, f.hom)
        is_linear_map(f.src, t.carrier, f.hom)
        linear_map_preimage_submodule_contains_eq(f.src, t, f.hom, x)
        module_hom_submodule_comap(f, t).contains(x) = t.contains(f.hom(x))
    }
}

/// If the image of an element lies in a target submodule, the element lies in its comap.
theorem module_hom_submodule_comap_contains_of_image_in[R: Ring, M: AddCommGroup, N: AddCommGroup](
    f: ModuleHom[R, M, N], t: Submodule[R, N], x: M
) {
    t.carrier = f.dst and t.contains(f.hom(x)) implies module_hom_submodule_comap(f, t).contains(x)
} by {
    if t.carrier = f.dst and t.contains(f.hom(x)) {
        module_hom_is_linear_map(f)
        is_linear_map(f.src, f.dst, f.hom)
        is_linear_map(f.src, t.carrier, f.hom)
        linear_map_preimage_submodule_contains_of_image_in(f.src, t, f.hom, x)
        module_hom_submodule_comap(f, t).contains(x)
    }
}

/// A member of a comap has image in the target submodule.
theorem module_hom_submodule_comap_image_in_of_contains[R: Ring, M: AddCommGroup, N: AddCommGroup](
    f: ModuleHom[R, M, N], t: Submodule[R, N], x: M
) {
    t.carrier = f.dst and module_hom_submodule_comap(f, t).contains(x) implies t.contains(f.hom(x))
} by {
    if t.carrier = f.dst and module_hom_submodule_comap(f, t).contains(x) {
        module_hom_is_linear_map(f)
        is_linear_map(f.src, f.dst, f.hom)
        is_linear_map(f.src, t.carrier, f.hom)
        linear_map_preimage_submodule_image_in_of_contains(f.src, t, f.hom, x)
        t.contains(f.hom(x))
    }
}

/// The bundled map/comap Galois connection for a module homomorphism.
theorem module_hom_submodule_map_subset_iff_comap[R: Ring, M: AddCommGroup, N: AddCommGroup](
    f: ModuleHom[R, M, N], s: Submodule[R, M], t: Submodule[R, N]
) {
    s.carrier = f.src and t.carrier = f.dst implies
        submodule_subset(module_hom_submodule_map(f, s), t) =
            submodule_subset(s, module_hom_submodule_comap(f, t))
} by {
    if s.carrier = f.src and t.carrier = f.dst {
        module_hom_is_linear_map(f)
        linear_map_submodule_image_subset_iff_preimage(f.src, f.dst, f.hom, s, t)
        submodule_subset(module_hom_submodule_map(f, s), t) =
            submodule_subset(s, module_hom_submodule_comap(f, t))
    }
}

/// The submodule map operation is monotone in its source submodule.
theorem module_hom_submodule_map_mono[R: Ring, M: AddCommGroup, N: AddCommGroup](
    f: ModuleHom[R, M, N], s1: Submodule[R, M], s2: Submodule[R, M]
) {
    s1.carrier = f.src and s2.carrier = f.src and submodule_subset(s1, s2)
        implies submodule_subset(module_hom_submodule_map(f, s1), module_hom_submodule_map(f, s2))
} by {
    if s1.carrier = f.src and s2.carrier = f.src and submodule_subset(s1, s2) {
        let image2 = module_hom_submodule_map(f, s2)
        module_hom_submodule_map_carrier_eq(f, s2)
        image2.carrier = f.dst
        forall(x: M) {
            if s1.contains(x) {
                submodule_subset(s1, s2) = forall(y: M) {
                    s1.contains(y) implies s2.contains(y)
                }
                s2.contains(x)
                module_hom_submodule_map_contains_apply(f, s2, x)
                image2.contains(f.hom(x))
                module_hom_submodule_comap_contains_of_image_in(f, image2, x)
                module_hom_submodule_comap(f, image2).contains(x)
            }
        }
        submodule_subset_intro(s1, module_hom_submodule_comap(f, image2))
        submodule_subset(s1, module_hom_submodule_comap(f, image2))
        module_hom_submodule_map_subset_iff_comap(f, s1, image2)
        submodule_subset(module_hom_submodule_map(f, s1), image2)
        submodule_subset(module_hom_submodule_map(f, s1), module_hom_submodule_map(f, s2))
    }
}

/// The submodule comap operation is monotone in its target submodule.
theorem module_hom_submodule_comap_mono[R: Ring, M: AddCommGroup, N: AddCommGroup](
    f: ModuleHom[R, M, N], t1: Submodule[R, N], t2: Submodule[R, N]
) {
    t1.carrier = f.dst and t2.carrier = f.dst and submodule_subset(t1, t2)
        implies submodule_subset(module_hom_submodule_comap(f, t1), module_hom_submodule_comap(f, t2))
} by {
    if t1.carrier = f.dst and t2.carrier = f.dst and submodule_subset(t1, t2) {
        module_hom_is_linear_map(f)
        is_linear_map(f.src, f.dst, f.hom)
        is_linear_map(f.src, t1.carrier, f.hom)
        t1.carrier = t2.carrier
        linear_map_preimage_submodule_mono(f.src, t1, t2, f.hom)
        submodule_subset(module_hom_submodule_comap(f, t1), module_hom_submodule_comap(f, t2))
    }
}

/// The comap of the full target submodule is the full source submodule.
theorem module_hom_submodule_comap_full_eq_full[R: Ring, M: AddCommGroup, N: AddCommGroup](
    f: ModuleHom[R, M, N]
) {
    module_hom_submodule_comap(f, full_submodule(f.dst)) = full_submodule(f.src)
} by {
    let comap_full = module_hom_submodule_comap(f, full_submodule(f.dst))
    let full_src = full_submodule(f.src)
    module_hom_is_linear_map(f)
    linear_map_preimage_submodule_full_subset(f.src, f.dst, f.hom)
    submodule_subset(comap_full, full_src)
    full_submodule_subset_linear_map_preimage_full(f.src, f.dst, f.hom)
    submodule_subset(full_src, comap_full)
    module_hom_submodule_comap_carrier_eq(f, full_submodule(f.dst))
    full_submodule_carrier_eq(f.src)
    comap_full.carrier = f.src
    full_src.carrier = f.src
    comap_full.carrier = full_src.carrier
    submodule_subset_antisymm(comap_full, full_src)
    comap_full = full_src
    module_hom_submodule_comap(f, full_submodule(f.dst)) = full_submodule(f.src)
}

/// The comap of the zero target submodule is the kernel submodule.
theorem module_hom_submodule_comap_zero_eq_kernel[R: Ring, M: AddCommGroup, N: AddCommGroup](
    f: ModuleHom[R, M, N]
) {
    module_hom_submodule_comap(f, zero_submodule(f.dst)) = module_hom_kernel(f)
} by {
    let comap_zero = module_hom_submodule_comap(f, zero_submodule(f.dst))
    let kernel = module_hom_kernel(f)
    module_hom_is_linear_map(f)
    linear_map_preimage_submodule_zero_subset_kernel(f.src, f.dst, f.hom)
    submodule_subset(comap_zero, linear_map_kernel_submodule(f.src, f.dst, f.hom))
    module_hom_kernel(f) = linear_map_kernel_submodule(f.src, f.dst, f.hom)
    submodule_subset(comap_zero, kernel)
    linear_map_kernel_subset_preimage_submodule_zero(f.src, f.dst, f.hom)
    submodule_subset(linear_map_kernel_submodule(f.src, f.dst, f.hom), comap_zero)
    submodule_subset(kernel, comap_zero)
    module_hom_submodule_comap_carrier_eq(f, zero_submodule(f.dst))
    module_hom_kernel_carrier(f)
    comap_zero.carrier = f.src
    kernel.carrier = f.src
    comap_zero.carrier = kernel.carrier
    submodule_subset_antisymm(comap_zero, kernel)
    comap_zero = kernel
    module_hom_submodule_comap(f, zero_submodule(f.dst)) = module_hom_kernel(f)
}

/// The map of any source submodule is contained in the image of the module homomorphism.
theorem module_hom_submodule_map_subset_image[R: Ring, M: AddCommGroup, N: AddCommGroup](
    f: ModuleHom[R, M, N], s: Submodule[R, M]
) {
    s.carrier = f.src implies submodule_subset(module_hom_submodule_map(f, s), module_hom_image(f))
} by {
    if s.carrier = f.src {
        let image = module_hom_image(f)
        module_hom_image_carrier(f)
        image.carrier = f.dst
        forall(x: M) {
            if s.contains(x) {
                module_hom_image_contains_value(f, x)
                image.contains(f.hom(x))
                module_hom_submodule_comap_contains_of_image_in(f, image, x)
                module_hom_submodule_comap(f, image).contains(x)
            }
        }
        submodule_subset_intro(s, module_hom_submodule_comap(f, image))
        submodule_subset(s, module_hom_submodule_comap(f, image))
        module_hom_submodule_map_subset_iff_comap(f, s, image)
        submodule_subset(module_hom_submodule_map(f, s), image)
        submodule_subset(module_hom_submodule_map(f, s), module_hom_image(f))
    }
}

/// The image of a module homomorphism is contained in the map of the full source submodule.
theorem module_hom_image_subset_submodule_map_full[R: Ring, M: AddCommGroup, N: AddCommGroup](
    f: ModuleHom[R, M, N]
) {
    submodule_subset(module_hom_image(f), module_hom_submodule_map(f, full_submodule(f.src)))
} by {
    forall(y: N) {
        if module_hom_image(f).contains(y) {
            module_hom_image_elim(f, y)
            let x: M satisfy { f.hom(x) = y }
            full_submodule_contains_everything(f.src, x)
            module_hom_submodule_map_contains_apply(f, full_submodule(f.src), x)
            module_hom_submodule_map(f, full_submodule(f.src)).contains(f.hom(x))
            module_hom_submodule_map(f, full_submodule(f.src)).contains(y)
        }
    }
}

/// Mapping the full source submodule gives exactly the image submodule of the homomorphism.
theorem module_hom_submodule_map_full_eq_image[R: Ring, M: AddCommGroup, N: AddCommGroup](
    f: ModuleHom[R, M, N]
) {
    module_hom_submodule_map(f, full_submodule(f.src)) = module_hom_image(f)
} by {
    let map_full = module_hom_submodule_map(f, full_submodule(f.src))
    let image = module_hom_image(f)
    full_submodule_carrier_eq(f.src)
    module_hom_submodule_map_subset_image(f, full_submodule(f.src))
    submodule_subset(map_full, image)
    module_hom_image_subset_submodule_map_full(f)
    submodule_subset(image, map_full)
    module_hom_submodule_map_carrier_eq(f, full_submodule(f.src))
    module_hom_image_carrier(f)
    map_full.carrier = f.dst
    image.carrier = f.dst
    map_full.carrier = image.carrier
    submodule_subset_antisymm(map_full, image)
    map_full = image
    module_hom_submodule_map(f, full_submodule(f.src)) = module_hom_image(f)
}
