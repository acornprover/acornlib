from algebra.group import Group, GroupHom
from algebra.monoid.monoid import Monoid, MonoidHom
from algebra.add_group import AddGroup, AddGroupHom
from algebra.ring.ring import Ring
from algebra.ring.ring_hom import RingHom
from semiring import Semiring
from algebra.semiring_hom import SemiringHom
from data.basic.functions import is_injective_fn, is_surjective_fn
from data.basic.set import Set, set_image, set_preimage, maps_into_set_image,
    set_preimage_contains_image, set_image_preimage_closure,
    set_preimage_image_kernel, set_image_preimage_closure_of_injective,
    set_preimage_image_kernel_of_surjective
from algebra.group.group_hom_set import group_hom_image, group_hom_preimage,
    group_hom_image_preimage_closure, group_hom_preimage_image_kernel,
    group_hom_preimage_image_kernel_reductive
from algebra.monoid.monoid_hom_set import monoid_hom_image, monoid_hom_preimage,
    monoid_hom_image_preimage_closure, monoid_hom_preimage_image_kernel,
    monoid_hom_preimage_image_kernel_reductive
from algebra.add_group_hom_set import add_group_hom_image, add_group_hom_preimage,
    add_group_hom_image_preimage_closure, add_group_hom_preimage_image_kernel,
    add_group_hom_preimage_image_kernel_reductive
from algebra.ring.ring_hom_set import ring_hom_image, ring_hom_preimage,
    ring_hom_image_preimage_closure, ring_hom_preimage_image_kernel,
    ring_hom_preimage_image_kernel_reductive
from algebra.ring.semiring_hom_set import semiring_hom_image, semiring_hom_preimage,
    semiring_hom_image_preimage_closure, semiring_hom_preimage_image_kernel,
    semiring_hom_preimage_image_kernel_reductive
from algebra.hom_kernel_injective import is_trivial_group_hom_kernel,
    group_hom_injective_iff_trivial_kernel, is_trivial_add_group_hom_kernel,
    add_group_hom_injective_iff_trivial_kernel, is_trivial_ring_hom_kernel,
    ring_hom_injective_iff_trivial_kernel

/// Membership in a group-hom preimage is membership of the image in the target set.
theorem group_hom_preimage_contains_eq[G: Group, H: Group](f: GroupHom[G, H], s: Set[H], x: G) {
    group_hom_preimage(f, s).contains(x) = s.contains(f.hom(x))
} by {
    set_preimage_contains_image(f.hom, s, x)
}

/// Membership in image-preimage closure is membership of the image in the hom image.
theorem group_hom_image_preimage_closure_contains_image_eq[G: Group, H: Group](
    f: GroupHom[G, H],
    s: Set[G],
    x: G
) {
    group_hom_image_preimage_closure(f, s).contains(x) =
        group_hom_image(f, s).contains(f.hom(x))
} by {
    set_image_preimage_closure(f.hom, s) = set_preimage(f.hom, set_image(s, f.hom))
    set_preimage_contains_image(f.hom, set_image(s, f.hom), x)
}

/// On actual image points, the preimage-image kernel is exactly the target set predicate.
theorem group_hom_preimage_image_kernel_at_image[G: Group, H: Group](
    f: GroupHom[G, H],
    s: Set[H],
    x: G
) {
    group_hom_preimage_image_kernel(f, s).contains(f.hom(x)) = s.contains(f.hom(x))
} by {
    if group_hom_preimage_image_kernel(f, s).contains(f.hom(x)) {
        let k = group_hom_preimage_image_kernel(f, s)
        group_hom_preimage_image_kernel_reductive(f, s)
        k.subset(s) = forall(y: H) {
            k.contains(y) implies s.contains(y)
        }
        k.contains(f.hom(x))
        s.contains(f.hom(x))
    }
    if s.contains(f.hom(x)) {
        set_preimage_contains_image(f.hom, s, x)
        maps_into_set_image(set_preimage(f.hom, s), f.hom, x)
        set_preimage_image_kernel(f.hom, s) = set_image(set_preimage(f.hom, s), f.hom)
        group_hom_preimage_image_kernel(f, s).contains(f.hom(x))
    }
}

/// Injective group homomorphisms have trivial image-preimage closure.
theorem group_hom_image_preimage_closure_of_injective[G: Group, H: Group](
    f: GroupHom[G, H],
    s: Set[G]
) {
    is_injective_fn(f.hom) implies group_hom_image_preimage_closure(f, s) = s
} by {
    if is_injective_fn(f.hom) {
        group_hom_image_preimage_closure(f, s) = set_image_preimage_closure(f.hom, s)
        set_image_preimage_closure_of_injective(f.hom, s)
    }
}

/// Surjective group homomorphisms have trivial preimage-image kernel.
theorem group_hom_preimage_image_kernel_of_surjective[G: Group, H: Group](
    f: GroupHom[G, H],
    s: Set[H]
) {
    is_surjective_fn(f.hom) implies group_hom_preimage_image_kernel(f, s) = s
} by {
    if is_surjective_fn(f.hom) {
        group_hom_preimage_image_kernel(f, s) = set_preimage_image_kernel(f.hom, s)
        set_preimage_image_kernel_of_surjective(f.hom, s)
    }
}

/// Membership in a monoid-hom preimage is membership of the image in the target set.
theorem monoid_hom_preimage_contains_eq[M: Monoid, N: Monoid](f: MonoidHom[M, N], s: Set[N], x: M) {
    monoid_hom_preimage(f, s).contains(x) = s.contains(f.hom(x))
} by {
    set_preimage_contains_image(f.hom, s, x)
}

/// Membership in image-preimage closure is membership of the image in the hom image.
theorem monoid_hom_image_preimage_closure_contains_image_eq[M: Monoid, N: Monoid](
    f: MonoidHom[M, N],
    s: Set[M],
    x: M
) {
    monoid_hom_image_preimage_closure(f, s).contains(x) =
        monoid_hom_image(f, s).contains(f.hom(x))
} by {
    set_image_preimage_closure(f.hom, s) = set_preimage(f.hom, set_image(s, f.hom))
    set_preimage_contains_image(f.hom, set_image(s, f.hom), x)
}

/// On actual image points, the preimage-image kernel is exactly the target set predicate.
theorem monoid_hom_preimage_image_kernel_at_image[M: Monoid, N: Monoid](
    f: MonoidHom[M, N],
    s: Set[N],
    x: M
) {
    monoid_hom_preimage_image_kernel(f, s).contains(f.hom(x)) = s.contains(f.hom(x))
} by {
    if monoid_hom_preimage_image_kernel(f, s).contains(f.hom(x)) {
        let k = monoid_hom_preimage_image_kernel(f, s)
        monoid_hom_preimage_image_kernel_reductive(f, s)
        k.subset(s) = forall(y: N) {
            k.contains(y) implies s.contains(y)
        }
        k.contains(f.hom(x))
        s.contains(f.hom(x))
    }
    if s.contains(f.hom(x)) {
        set_preimage_contains_image(f.hom, s, x)
        maps_into_set_image(set_preimage(f.hom, s), f.hom, x)
        set_preimage_image_kernel(f.hom, s) = set_image(set_preimage(f.hom, s), f.hom)
        monoid_hom_preimage_image_kernel(f, s).contains(f.hom(x))
    }
}

/// Injective monoid homomorphisms have trivial image-preimage closure.
theorem monoid_hom_image_preimage_closure_of_injective[M: Monoid, N: Monoid](
    f: MonoidHom[M, N],
    s: Set[M]
) {
    is_injective_fn(f.hom) implies monoid_hom_image_preimage_closure(f, s) = s
} by {
    if is_injective_fn(f.hom) {
        monoid_hom_image_preimage_closure(f, s) = set_image_preimage_closure(f.hom, s)
        set_image_preimage_closure_of_injective(f.hom, s)
    }
}

/// Surjective monoid homomorphisms have trivial preimage-image kernel.
theorem monoid_hom_preimage_image_kernel_of_surjective[M: Monoid, N: Monoid](
    f: MonoidHom[M, N],
    s: Set[N]
) {
    is_surjective_fn(f.hom) implies monoid_hom_preimage_image_kernel(f, s) = s
} by {
    if is_surjective_fn(f.hom) {
        monoid_hom_preimage_image_kernel(f, s) = set_preimage_image_kernel(f.hom, s)
        set_preimage_image_kernel_of_surjective(f.hom, s)
    }
}

/// Membership in an additive-group-hom preimage is membership of the image in the target set.
theorem add_group_hom_preimage_contains_eq[A: AddGroup, B: AddGroup](f: AddGroupHom[A, B], s: Set[B], x: A) {
    add_group_hom_preimage(f, s).contains(x) = s.contains(f.hom(x))
} by {
    set_preimage_contains_image(f.hom, s, x)
}

/// Membership in image-preimage closure is membership of the image in the hom image.
theorem add_group_hom_image_preimage_closure_contains_image_eq[A: AddGroup, B: AddGroup](
    f: AddGroupHom[A, B],
    s: Set[A],
    x: A
) {
    add_group_hom_image_preimage_closure(f, s).contains(x) =
        add_group_hom_image(f, s).contains(f.hom(x))
} by {
    set_image_preimage_closure(f.hom, s) = set_preimage(f.hom, set_image(s, f.hom))
    set_preimage_contains_image(f.hom, set_image(s, f.hom), x)
}

/// On actual image points, the preimage-image kernel is exactly the target set predicate.
theorem add_group_hom_preimage_image_kernel_at_image[A: AddGroup, B: AddGroup](
    f: AddGroupHom[A, B],
    s: Set[B],
    x: A
) {
    add_group_hom_preimage_image_kernel(f, s).contains(f.hom(x)) = s.contains(f.hom(x))
} by {
    if add_group_hom_preimage_image_kernel(f, s).contains(f.hom(x)) {
        let k = add_group_hom_preimage_image_kernel(f, s)
        add_group_hom_preimage_image_kernel_reductive(f, s)
        k.subset(s) = forall(y: B) {
            k.contains(y) implies s.contains(y)
        }
        k.contains(f.hom(x))
        s.contains(f.hom(x))
    }
    if s.contains(f.hom(x)) {
        set_preimage_contains_image(f.hom, s, x)
        maps_into_set_image(set_preimage(f.hom, s), f.hom, x)
        set_preimage_image_kernel(f.hom, s) = set_image(set_preimage(f.hom, s), f.hom)
        add_group_hom_preimage_image_kernel(f, s).contains(f.hom(x))
    }
}

/// Injective additive group homomorphisms have trivial image-preimage closure.
theorem add_group_hom_image_preimage_closure_of_injective[A: AddGroup, B: AddGroup](
    f: AddGroupHom[A, B],
    s: Set[A]
) {
    is_injective_fn(f.hom) implies add_group_hom_image_preimage_closure(f, s) = s
} by {
    if is_injective_fn(f.hom) {
        add_group_hom_image_preimage_closure(f, s) = set_image_preimage_closure(f.hom, s)
        set_image_preimage_closure_of_injective(f.hom, s)
    }
}

/// Surjective additive group homomorphisms have trivial preimage-image kernel.
theorem add_group_hom_preimage_image_kernel_of_surjective[A: AddGroup, B: AddGroup](
    f: AddGroupHom[A, B],
    s: Set[B]
) {
    is_surjective_fn(f.hom) implies add_group_hom_preimage_image_kernel(f, s) = s
} by {
    if is_surjective_fn(f.hom) {
        add_group_hom_preimage_image_kernel(f, s) = set_preimage_image_kernel(f.hom, s)
        set_preimage_image_kernel_of_surjective(f.hom, s)
    }
}

/// Membership in a ring-hom preimage is membership of the image in the target set.
theorem ring_hom_preimage_contains_eq[R: Ring, S: Ring](f: RingHom[R, S], s: Set[S], x: R) {
    ring_hom_preimage(f, s).contains(x) = s.contains(f.hom(x))
} by {
    set_preimage_contains_image(f.hom, s, x)
}

/// Membership in image-preimage closure is membership of the image in the hom image.
theorem ring_hom_image_preimage_closure_contains_image_eq[R: Ring, S: Ring](
    f: RingHom[R, S],
    s: Set[R],
    x: R
) {
    ring_hom_image_preimage_closure(f, s).contains(x) =
        ring_hom_image(f, s).contains(f.hom(x))
} by {
    set_image_preimage_closure(f.hom, s) = set_preimage(f.hom, set_image(s, f.hom))
    set_preimage_contains_image(f.hom, set_image(s, f.hom), x)
}

/// On actual image points, the preimage-image kernel is exactly the target set predicate.
theorem ring_hom_preimage_image_kernel_at_image[R: Ring, S: Ring](
    f: RingHom[R, S],
    s: Set[S],
    x: R
) {
    ring_hom_preimage_image_kernel(f, s).contains(f.hom(x)) = s.contains(f.hom(x))
} by {
    if ring_hom_preimage_image_kernel(f, s).contains(f.hom(x)) {
        let k = ring_hom_preimage_image_kernel(f, s)
        ring_hom_preimage_image_kernel_reductive(f, s)
        k.subset(s) = forall(y: S) {
            k.contains(y) implies s.contains(y)
        }
        k.contains(f.hom(x))
        s.contains(f.hom(x))
    }
    if s.contains(f.hom(x)) {
        set_preimage_contains_image(f.hom, s, x)
        maps_into_set_image(set_preimage(f.hom, s), f.hom, x)
        set_preimage_image_kernel(f.hom, s) = set_image(set_preimage(f.hom, s), f.hom)
        ring_hom_preimage_image_kernel(f, s).contains(f.hom(x))
    }
}

/// Injective ring homomorphisms have trivial image-preimage closure.
theorem ring_hom_image_preimage_closure_of_injective[R: Ring, S: Ring](
    f: RingHom[R, S],
    s: Set[R]
) {
    is_injective_fn(f.hom) implies ring_hom_image_preimage_closure(f, s) = s
} by {
    if is_injective_fn(f.hom) {
        ring_hom_image_preimage_closure(f, s) = set_image_preimage_closure(f.hom, s)
        set_image_preimage_closure_of_injective(f.hom, s)
    }
}

/// Surjective ring homomorphisms have trivial preimage-image kernel.
theorem ring_hom_preimage_image_kernel_of_surjective[R: Ring, S: Ring](
    f: RingHom[R, S],
    s: Set[S]
) {
    is_surjective_fn(f.hom) implies ring_hom_preimage_image_kernel(f, s) = s
} by {
    if is_surjective_fn(f.hom) {
        ring_hom_preimage_image_kernel(f, s) = set_preimage_image_kernel(f.hom, s)
        set_preimage_image_kernel_of_surjective(f.hom, s)
    }
}

/// Membership in a semiring-hom preimage is membership of the image in the target set.
theorem semiring_hom_preimage_contains_eq[R: Semiring, S: Semiring](f: SemiringHom[R, S], s: Set[S], x: R) {
    semiring_hom_preimage(f, s).contains(x) = s.contains(f.hom(x))
} by {
    set_preimage_contains_image(f.hom, s, x)
}

/// Membership in image-preimage closure is membership of the image in the hom image.
theorem semiring_hom_image_preimage_closure_contains_image_eq[R: Semiring, S: Semiring](
    f: SemiringHom[R, S],
    s: Set[R],
    x: R
) {
    semiring_hom_image_preimage_closure(f, s).contains(x) =
        semiring_hom_image(f, s).contains(f.hom(x))
} by {
    set_image_preimage_closure(f.hom, s) = set_preimage(f.hom, set_image(s, f.hom))
    set_preimage_contains_image(f.hom, set_image(s, f.hom), x)
}

/// On actual image points, the preimage-image kernel is exactly the target set predicate.
theorem semiring_hom_preimage_image_kernel_at_image[R: Semiring, S: Semiring](
    f: SemiringHom[R, S],
    s: Set[S],
    x: R
) {
    semiring_hom_preimage_image_kernel(f, s).contains(f.hom(x)) = s.contains(f.hom(x))
} by {
    if semiring_hom_preimage_image_kernel(f, s).contains(f.hom(x)) {
        let k = semiring_hom_preimage_image_kernel(f, s)
        semiring_hom_preimage_image_kernel_reductive(f, s)
        k.subset(s) = forall(y: S) {
            k.contains(y) implies s.contains(y)
        }
        k.contains(f.hom(x))
        s.contains(f.hom(x))
    }
    if s.contains(f.hom(x)) {
        set_preimage_contains_image(f.hom, s, x)
        maps_into_set_image(set_preimage(f.hom, s), f.hom, x)
        set_preimage_image_kernel(f.hom, s) = set_image(set_preimage(f.hom, s), f.hom)
        semiring_hom_preimage_image_kernel(f, s).contains(f.hom(x))
    }
}

/// Injective semiring homomorphisms have trivial image-preimage closure.
theorem semiring_hom_image_preimage_closure_of_injective[R: Semiring, S: Semiring](
    f: SemiringHom[R, S],
    s: Set[R]
) {
    is_injective_fn(f.hom) implies semiring_hom_image_preimage_closure(f, s) = s
} by {
    if is_injective_fn(f.hom) {
        semiring_hom_image_preimage_closure(f, s) = set_image_preimage_closure(f.hom, s)
        set_image_preimage_closure_of_injective(f.hom, s)
    }
}

/// Surjective semiring homomorphisms have trivial preimage-image kernel.
theorem semiring_hom_preimage_image_kernel_of_surjective[R: Semiring, S: Semiring](
    f: SemiringHom[R, S],
    s: Set[S]
) {
    is_surjective_fn(f.hom) implies semiring_hom_preimage_image_kernel(f, s) = s
} by {
    if is_surjective_fn(f.hom) {
        semiring_hom_preimage_image_kernel(f, s) = set_preimage_image_kernel(f.hom, s)
        set_preimage_image_kernel_of_surjective(f.hom, s)
    }
}

/// Injective group homomorphisms have trivial kernel.
theorem group_hom_trivial_kernel_of_injective[G: Group, H: Group](f: GroupHom[G, H]) {
    is_injective_fn(f.hom) implies is_trivial_group_hom_kernel(f)
} by {
    if is_injective_fn(f.hom) {
        group_hom_injective_iff_trivial_kernel(f)
        is_trivial_group_hom_kernel(f)
    }
}

/// Trivial kernel implies injectivity for group homomorphisms.
theorem group_hom_injective_of_trivial_kernel[G: Group, H: Group](f: GroupHom[G, H]) {
    is_trivial_group_hom_kernel(f) implies is_injective_fn(f.hom)
} by {
    if is_trivial_group_hom_kernel(f) {
        group_hom_injective_iff_trivial_kernel(f)
        is_injective_fn(f.hom)
    }
}

/// Injectivity turns kernel membership into equality with the identity element.
theorem group_hom_kernel_element_eq_one_of_injective[G: Group, H: Group](f: GroupHom[G, H], a: G) {
    is_injective_fn(f.hom) and f.hom(a) = H.1 implies a = G.1
} by {
    if is_injective_fn(f.hom) and f.hom(a) = H.1 {
        group_hom_trivial_kernel_of_injective(f)
        is_trivial_group_hom_kernel(f) = forall(x: G) {
            f.hom(x) = H.1 implies x = G.1
        }
        a = G.1
    }
}

/// Injective additive group homomorphisms have trivial kernel.
theorem add_group_hom_trivial_kernel_of_injective[A: AddGroup, B: AddGroup](f: AddGroupHom[A, B]) {
    is_injective_fn(f.hom) implies is_trivial_add_group_hom_kernel(f)
} by {
    if is_injective_fn(f.hom) {
        add_group_hom_injective_iff_trivial_kernel(f)
        is_trivial_add_group_hom_kernel(f)
    }
}

/// Trivial kernel implies injectivity for additive group homomorphisms.
theorem add_group_hom_injective_of_trivial_kernel[A: AddGroup, B: AddGroup](f: AddGroupHom[A, B]) {
    is_trivial_add_group_hom_kernel(f) implies is_injective_fn(f.hom)
} by {
    if is_trivial_add_group_hom_kernel(f) {
        add_group_hom_injective_iff_trivial_kernel(f)
        is_injective_fn(f.hom)
    }
}

/// Injectivity turns additive kernel membership into equality with zero.
theorem add_group_hom_kernel_element_eq_zero_of_injective[A: AddGroup, B: AddGroup](f: AddGroupHom[A, B], a: A) {
    is_injective_fn(f.hom) and f.hom(a) = B.0 implies a = A.0
} by {
    if is_injective_fn(f.hom) and f.hom(a) = B.0 {
        add_group_hom_trivial_kernel_of_injective(f)
        is_trivial_add_group_hom_kernel(f) = forall(x: A) {
            f.hom(x) = B.0 implies x = A.0
        }
        a = A.0
    }
}

/// Injective ring homomorphisms have trivial kernel.
theorem ring_hom_trivial_kernel_of_injective[R: Ring, S: Ring](f: RingHom[R, S]) {
    is_injective_fn(f.hom) implies is_trivial_ring_hom_kernel(f)
} by {
    if is_injective_fn(f.hom) {
        ring_hom_injective_iff_trivial_kernel(f)
        is_trivial_ring_hom_kernel(f)
    }
}

/// Trivial kernel implies injectivity for ring homomorphisms.
theorem ring_hom_injective_of_trivial_kernel[R: Ring, S: Ring](f: RingHom[R, S]) {
    is_trivial_ring_hom_kernel(f) implies is_injective_fn(f.hom)
} by {
    if is_trivial_ring_hom_kernel(f) {
        ring_hom_injective_iff_trivial_kernel(f)
        is_injective_fn(f.hom)
    }
}

/// Injectivity turns ring kernel membership into equality with zero.
theorem ring_hom_kernel_element_eq_zero_of_injective[R: Ring, S: Ring](f: RingHom[R, S], a: R) {
    is_injective_fn(f.hom) and f.hom(a) = S.0 implies a = R.0
} by {
    if is_injective_fn(f.hom) and f.hom(a) = S.0 {
        ring_hom_trivial_kernel_of_injective(f)
        is_trivial_ring_hom_kernel(f) = forall(x: R) {
            f.hom(x) = S.0 implies x = R.0
        }
        a = R.0
    }
}
