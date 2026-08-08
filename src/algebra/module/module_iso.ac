from algebra.ring.ring import Ring
from algebra.add_comm_group import AddCommGroup
from data.basic.functions import identity_fn, compose, is_two_sided_inverse_fn,
    is_left_inverse_fn, is_right_inverse_fn, two_sided_inverse_fn_symm,
    compose_two_sided_inverse_fn, two_sided_inverse_fn_imp_bijection_fn,
    is_bijection_fn, two_sided_inverse_fn_left_apply, two_sided_inverse_fn_right_apply,
    is_injective_fn, is_surjective_fn
from algebra.module.module import Module
from algebra.module.module_hom import is_linear_equiv,
    linear_equiv_two_sided_inverse_is_linear_equiv,
    linear_equiv_is_linear_map, is_linear_map, identity_fn_is_linear_map,
    linear_map_two_sided_inverse_is_linear_map, linear_map_zero, linear_map_add,
    linear_map_smul, linear_map_neg, linear_map_sub, compose_is_linear_map,
    ModuleHom, module_hom_is_linear_map,
    linear_equiv_is_injective, linear_equiv_is_surjective,
    linear_equiv_is_bijection,
    linear_equiv_two_sided_inverse_zero, linear_equiv_two_sided_inverse_add,
    linear_equiv_two_sided_inverse_smul, linear_equiv_two_sided_inverse_neg,
    linear_equiv_two_sided_inverse_sub, linear_equiv_apply_eq_iff,
    linear_equiv_apply_eq_zero_iff_eq_zero

/// True if f is an R-linear equivalence from the source module to the destination
/// module with two-sided inverse g, mirroring the bundling of a forward and inverse
/// map in a linear equivalence.
define is_linear_equiv_pair[R: Ring, M: AddCommGroup, N: AddCommGroup](
    src: Module[R, M], dst: Module[R, N], f: M -> N, g: N -> M) -> Bool {
    is_linear_map(src, dst, f) and is_two_sided_inverse_fn(f, g)
}

/// The forward map of a linear equivalence pair is a linear equivalence.
theorem linear_equiv_pair_forward[R: Ring, M: AddCommGroup, N: AddCommGroup](
    src: Module[R, M], dst: Module[R, N], f: M -> N, g: N -> M) {
    is_linear_equiv_pair(src, dst, f, g) implies is_linear_equiv(src, dst, f)
} by {
    if is_linear_equiv_pair(src, dst, f, g) {
        is_linear_equiv_pair(src, dst, f, g) =
            is_linear_map(src, dst, f) and is_two_sided_inverse_fn(f, g)
        two_sided_inverse_fn_imp_bijection_fn(f, g)
        is_bijection_fn(f)
        is_linear_equiv(src, dst, f)
    }
}

/// The maps of a linear equivalence pair are two-sided inverses.
theorem linear_equiv_pair_inverse[R: Ring, M: AddCommGroup, N: AddCommGroup](
    src: Module[R, M], dst: Module[R, N], f: M -> N, g: N -> M) {
    is_linear_equiv_pair(src, dst, f, g) implies is_two_sided_inverse_fn(f, g)
} by {
    if is_linear_equiv_pair(src, dst, f, g) {
        is_two_sided_inverse_fn(f, g)
    }
}

/// The inverse map of a linear equivalence pair is itself a linear equivalence in
/// the opposite direction.
theorem linear_equiv_pair_inverse_is_linear_equiv[R: Ring, M: AddCommGroup, N: AddCommGroup](
    src: Module[R, M], dst: Module[R, N], f: M -> N, g: N -> M) {
    is_linear_equiv_pair(src, dst, f, g) implies is_linear_equiv(dst, src, g)
} by {
    if is_linear_equiv_pair(src, dst, f, g) {
        is_linear_equiv_pair(src, dst, f, g) =
            is_linear_map(src, dst, f) and is_two_sided_inverse_fn(f, g)
        two_sided_inverse_fn_imp_bijection_fn(f, g)
        is_bijection_fn(f)
        linear_equiv_two_sided_inverse_is_linear_equiv(src, dst, f, g)
        is_linear_equiv(dst, src, g)
    }
}

/// The identity map paired with itself is a linear equivalence pair from a module to
/// itself.
theorem linear_equiv_pair_identity[R: Ring, M: AddCommGroup](src: Module[R, M]) {
    is_linear_equiv_pair(src, src, identity_fn[M], identity_fn[M])
} by {
    identity_fn_is_linear_map(src)
    forall(x: M) {
        identity_fn[M](identity_fn[M](x)) = x
    }
    is_left_inverse_fn(identity_fn[M], identity_fn[M])
    is_right_inverse_fn(identity_fn[M], identity_fn[M])
    is_two_sided_inverse_fn(identity_fn[M], identity_fn[M])
}

/// Reversing a linear equivalence pair gives a linear equivalence pair in the
/// opposite direction.
theorem linear_equiv_pair_swap[R: Ring, M: AddCommGroup, N: AddCommGroup](
    src: Module[R, M], dst: Module[R, N], f: M -> N, g: N -> M) {
    is_linear_equiv_pair(src, dst, f, g) implies is_linear_equiv_pair(dst, src, g, f)
} by {
    if is_linear_equiv_pair(src, dst, f, g) {
        linear_map_two_sided_inverse_is_linear_map(src, dst, f, g)
        is_linear_map(dst, src, g)
        two_sided_inverse_fn_symm(f, g)
        is_two_sided_inverse_fn(g, f)
        is_linear_equiv_pair(dst, src, g, f)
    }
}

/// Composing linear equivalence pairs gives a linear equivalence pair.
theorem linear_equiv_pair_compose[R: Ring, M: AddCommGroup, N: AddCommGroup, K: AddCommGroup](
    a: Module[R, M], b: Module[R, N], c: Module[R, K],
    f: N -> K, g: K -> N, h: M -> N, k: N -> M
) {
    is_linear_equiv_pair(b, c, f, g) and is_linear_equiv_pair(a, b, h, k)
    implies is_linear_equiv_pair(a, c, compose(f, h), compose(k, g))
} by {
    if is_linear_equiv_pair(b, c, f, g) and is_linear_equiv_pair(a, b, h, k) {
        is_linear_equiv_pair(b, c, f, g) =
            is_linear_map(b, c, f) and is_two_sided_inverse_fn(f, g)
        compose_is_linear_map(a, b, c, f, h)
        is_linear_map(a, c, compose(f, h))
        compose_two_sided_inverse_fn(h, f, k, g)
        is_two_sided_inverse_fn(compose(f, h), compose(k, g))
        is_linear_equiv_pair(a, c, compose(f, h), compose(k, g))
    }
}

/// True if the two R-modules are linearly equivalent, meaning there is a linear
/// equivalence between them with a two-sided inverse.
define modules_linearly_equivalent[R: Ring, M: AddCommGroup, N: AddCommGroup](
    src: Module[R, M], dst: Module[R, N]) -> Bool {
    exists(f: M -> N) {
        exists(g: N -> M) {
            is_linear_equiv_pair(src, dst, f, g)
        }
    }
}

/// A linear equivalence pair witnesses that two modules are linearly equivalent.
theorem modules_linearly_equivalent_of_pair[R: Ring, M: AddCommGroup, N: AddCommGroup](
    src: Module[R, M], dst: Module[R, N], f: M -> N, g: N -> M) {
    is_linear_equiv_pair(src, dst, f, g) implies modules_linearly_equivalent(src, dst)
} by {
    if is_linear_equiv_pair(src, dst, f, g) {
        modules_linearly_equivalent(src, dst)
    }
}

/// Linear equivalence of modules is reflexive.
theorem modules_linearly_equivalent_refl[R: Ring, M: AddCommGroup](src: Module[R, M]) {
    modules_linearly_equivalent(src, src)
} by {
    linear_equiv_pair_identity(src)
    modules_linearly_equivalent_of_pair(src, src, identity_fn[M], identity_fn[M])
}

/// Linear equivalence of modules is symmetric.
theorem modules_linearly_equivalent_symm[R: Ring, M: AddCommGroup, N: AddCommGroup](
    src: Module[R, M], dst: Module[R, N]) {
    modules_linearly_equivalent(src, dst) implies modules_linearly_equivalent(dst, src)
} by {
    if modules_linearly_equivalent(src, dst) {
        let f: M -> N satisfy {
            exists(g: N -> M) {
                is_linear_equiv_pair(src, dst, f, g)
            }
        }
        let g: N -> M satisfy {
            is_linear_equiv_pair(src, dst, f, g)
        }
        linear_equiv_pair_swap(src, dst, f, g)
        is_linear_equiv_pair(dst, src, g, f)
    }
}

/// Composing linear equivalence pairs witnesses that the outer modules are linearly
/// equivalent.
theorem modules_linearly_equivalent_of_composed_pairs[R: Ring, M: AddCommGroup, N: AddCommGroup, K: AddCommGroup](
    a: Module[R, M], b: Module[R, N], c: Module[R, K],
    f: N -> K, g: K -> N, h: M -> N, k: N -> M
) {
    is_linear_equiv_pair(b, c, f, g) and is_linear_equiv_pair(a, b, h, k)
    implies modules_linearly_equivalent(a, c)
} by {
    if is_linear_equiv_pair(b, c, f, g) and is_linear_equiv_pair(a, b, h, k) {
        linear_equiv_pair_compose(a, b, c, f, g, h, k)
        modules_linearly_equivalent(a, c)
    }
}

/// A linear equivalence pair from the first module composes with an equivalence of
/// the second and third modules to give an equivalence of the first and third.
theorem modules_linearly_equivalent_of_pair_trans[R: Ring, M: AddCommGroup, N: AddCommGroup, K: AddCommGroup](
    a: Module[R, M], b: Module[R, N], c: Module[R, K], h: M -> N, k: N -> M
) {
    is_linear_equiv_pair(a, b, h, k) and modules_linearly_equivalent(b, c)
    implies modules_linearly_equivalent(a, c)
} by {
    if is_linear_equiv_pair(a, b, h, k) and modules_linearly_equivalent(b, c) {
        let f: N -> K satisfy {
            exists(g: K -> N) {
                is_linear_equiv_pair(b, c, f, g)
            }
        }
        let g: K -> N satisfy {
            is_linear_equiv_pair(b, c, f, g)
        }
        modules_linearly_equivalent_of_composed_pairs(a, b, c, f, g, h, k)
        modules_linearly_equivalent(a, c)
    }
}

/// Linear equivalence of modules is transitive.
theorem modules_linearly_equivalent_trans[R: Ring, M: AddCommGroup, N: AddCommGroup, K: AddCommGroup](
    a: Module[R, M], b: Module[R, N], c: Module[R, K]) {
    modules_linearly_equivalent(a, b) and modules_linearly_equivalent(b, c)
    implies modules_linearly_equivalent(a, c)
} by {
    if modules_linearly_equivalent(a, b) and modules_linearly_equivalent(b, c) {
        let h: M -> N satisfy {
            exists(k: N -> M) {
                is_linear_equiv_pair(a, b, h, k)
            }
        }
        let k: N -> M satisfy {
            is_linear_equiv_pair(a, b, h, k)
        }
        modules_linearly_equivalent_of_pair_trans(a, b, c, h, k)
        modules_linearly_equivalent(a, c)
    }
}

/// A bundled R-linear equivalence between R-modules: a linear forward map together
/// with a two-sided inverse. The constraint is forall-only (it does not unfold to an
/// existential), so the constrained function fields extract without timing out.
structure LinearEquiv[R: Ring, M: AddCommGroup, N: AddCommGroup] {
    /// The source R-module.
    src: Module[R, M]
    /// The destination R-module.
    dst: Module[R, N]
    /// The underlying forward function.
    to_fn: M -> N
    /// The underlying inverse function.
    inv_fn: N -> M
} constraint {
    is_linear_map(src, dst, to_fn) and is_two_sided_inverse_fn(to_fn, inv_fn)
}

/// The forward function of a linear equivalence is a linear map.
theorem linear_equiv_to_fn_is_linear_map[R: Ring, M: AddCommGroup, N: AddCommGroup](
    e: LinearEquiv[R, M, N]
) {
    is_linear_map(e.src, e.dst, e.to_fn)
}

/// The forward and inverse functions of a linear equivalence are two-sided inverses.
theorem linear_equiv_to_inv_two_sided[R: Ring, M: AddCommGroup, N: AddCommGroup](
    e: LinearEquiv[R, M, N]
) {
    is_two_sided_inverse_fn(e.to_fn, e.inv_fn)
}

/// The inverse function precomposed with the forward function is the identity.
theorem linear_equiv_left_inverse[R: Ring, M: AddCommGroup, N: AddCommGroup](
    e: LinearEquiv[R, M, N], x: M
) {
    e.inv_fn(e.to_fn(x)) = x
} by {
    linear_equiv_to_inv_two_sided(e)
    two_sided_inverse_fn_left_apply(e.to_fn, e.inv_fn, x)
}

/// The forward function precomposed with the inverse function is the identity.
theorem linear_equiv_right_inverse[R: Ring, M: AddCommGroup, N: AddCommGroup](
    e: LinearEquiv[R, M, N], y: N
) {
    e.to_fn(e.inv_fn(y)) = y
} by {
    linear_equiv_to_inv_two_sided(e)
    two_sided_inverse_fn_right_apply(e.to_fn, e.inv_fn, y)
}

/// The forward function of a linear equivalence is a linear equivalence in the
/// predicate sense.
theorem linear_equiv_to_fn_is_linear_equiv[R: Ring, M: AddCommGroup, N: AddCommGroup](
    e: LinearEquiv[R, M, N]
) {
    is_linear_equiv(e.src, e.dst, e.to_fn)
} by {
    linear_equiv_to_fn_is_linear_map(e)
    linear_equiv_to_inv_two_sided(e)
    two_sided_inverse_fn_imp_bijection_fn(e.to_fn, e.inv_fn)
}

/// The inverse function of a linear equivalence is itself a linear map in the opposite
/// direction.
theorem linear_equiv_inv_fn_is_linear_map[R: Ring, M: AddCommGroup, N: AddCommGroup](
    e: LinearEquiv[R, M, N]
) {
    is_linear_map(e.dst, e.src, e.inv_fn)
} by {
    linear_equiv_to_fn_is_linear_map(e)
    linear_equiv_to_inv_two_sided(e)
    linear_map_two_sided_inverse_is_linear_map(e.src, e.dst, e.to_fn, e.inv_fn)
}

/// The forward and inverse functions of a linear equivalence form a linear
/// equivalence pair.
theorem linear_equiv_is_pair[R: Ring, M: AddCommGroup, N: AddCommGroup](
    e: LinearEquiv[R, M, N]
) {
    is_linear_equiv_pair(e.src, e.dst, e.to_fn, e.inv_fn)
} by {
    linear_equiv_to_fn_is_linear_map(e)
    linear_equiv_to_inv_two_sided(e)
}

/// A bundled linear equivalence witnesses that its source and destination modules are
/// linearly equivalent.
theorem linear_equiv_modules_linearly_equivalent[R: Ring, M: AddCommGroup, N: AddCommGroup](
    e: LinearEquiv[R, M, N]
) {
    modules_linearly_equivalent(e.src, e.dst)
} by {
    linear_equiv_is_pair(e)
    modules_linearly_equivalent_of_pair(e.src, e.dst, e.to_fn, e.inv_fn)
}

/// Linear equivalence extensionality: two linear equivalences with the same source,
/// destination, forward map, and inverse map are equal.
theorem linear_equiv_ext[R: Ring, M: AddCommGroup, N: AddCommGroup](
    e: LinearEquiv[R, M, N], h: LinearEquiv[R, M, N]
) {
    e.src = h.src and e.dst = h.dst and e.to_fn = h.to_fn and e.inv_fn = h.inv_fn
        implies e = h
}

/// A linear equivalence sends zero to zero.
theorem linear_equiv_map_zero[R: Ring, M: AddCommGroup, N: AddCommGroup](
    e: LinearEquiv[R, M, N]
) {
    e.to_fn(M.0) = N.0
} by {
    linear_equiv_to_fn_is_linear_map(e)
    linear_map_zero(e.src, e.dst, e.to_fn)
}

/// A linear equivalence preserves addition.
theorem linear_equiv_map_add[R: Ring, M: AddCommGroup, N: AddCommGroup](
    e: LinearEquiv[R, M, N], x: M, y: M
) {
    e.to_fn(x + y) = e.to_fn(x) + e.to_fn(y)
} by {
    linear_equiv_to_fn_is_linear_map(e)
    linear_map_add(e.src, e.dst, e.to_fn, x, y)
}

/// A linear equivalence preserves the scalar action.
theorem linear_equiv_map_smul[R: Ring, M: AddCommGroup, N: AddCommGroup](
    e: LinearEquiv[R, M, N], r: R, x: M
) {
    e.to_fn(e.src.smul(r, x)) = e.dst.smul(r, e.to_fn(x))
} by {
    linear_equiv_to_fn_is_linear_map(e)
    linear_map_smul(e.src, e.dst, e.to_fn, r, x)
}

/// A linear equivalence preserves negation.
theorem linear_equiv_map_neg[R: Ring, M: AddCommGroup, N: AddCommGroup](
    e: LinearEquiv[R, M, N], x: M
) {
    e.to_fn(-x) = -e.to_fn(x)
} by {
    linear_equiv_to_fn_is_linear_map(e)
    linear_map_neg(e.src, e.dst, e.to_fn, x)
}

/// A linear equivalence preserves subtraction.
theorem linear_equiv_map_sub[R: Ring, M: AddCommGroup, N: AddCommGroup](
    e: LinearEquiv[R, M, N], x: M, y: M
) {
    e.to_fn(x - y) = e.to_fn(x) - e.to_fn(y)
} by {
    linear_equiv_to_fn_is_linear_map(e)
    linear_map_sub(e.src, e.dst, e.to_fn, x, y)
}

/// The identity function is its own two-sided inverse.
theorem identity_fn_two_sided_inverse_self[M: AddCommGroup] {
    is_two_sided_inverse_fn(identity_fn[M], identity_fn[M])
} by {
    forall(x: M) {
        identity_fn[M](identity_fn[M](x)) = x
    }
    is_left_inverse_fn(identity_fn[M], identity_fn[M])
    is_right_inverse_fn(identity_fn[M], identity_fn[M])
}

/// The identity linear equivalence on an R-module.
let linear_equiv_identity[R: Ring, M: AddCommGroup](m: Module[R, M]) -> result: LinearEquiv[R, M, M] satisfy {
    LinearEquiv.new(m, m, identity_fn[M], identity_fn[M]) = Option.some(result)
} by {
    identity_fn_is_linear_map(m)
    identity_fn_two_sided_inverse_self[M]
    is_linear_map(m, m, identity_fn[M]) and is_two_sided_inverse_fn(identity_fn[M], identity_fn[M])
}

/// The identity linear equivalence has the given module as source.
theorem linear_equiv_identity_src[R: Ring, M: AddCommGroup](m: Module[R, M]) {
    linear_equiv_identity(m).src = m
} by {
    LinearEquiv.new(m, m, identity_fn[M], identity_fn[M]) = Option.some(linear_equiv_identity(m))
}

/// The identity linear equivalence has the given module as destination.
theorem linear_equiv_identity_dst[R: Ring, M: AddCommGroup](m: Module[R, M]) {
    linear_equiv_identity(m).dst = m
} by {
    LinearEquiv.new(m, m, identity_fn[M], identity_fn[M]) = Option.some(linear_equiv_identity(m))
}

/// The inverse of a linear equivalence, obtained by swapping the forward and inverse
/// maps.
let linear_equiv_symm[R: Ring, M: AddCommGroup, N: AddCommGroup](e: LinearEquiv[R, M, N]) -> result: LinearEquiv[R, N, M] satisfy {
    LinearEquiv.new(e.dst, e.src, e.inv_fn, e.to_fn) = Option.some(result)
} by {
    linear_equiv_inv_fn_is_linear_map(e)
    linear_equiv_to_inv_two_sided(e)
    two_sided_inverse_fn_symm(e.to_fn, e.inv_fn)
    is_linear_map(e.dst, e.src, e.inv_fn) and is_two_sided_inverse_fn(e.inv_fn, e.to_fn)
}

/// The inverse linear equivalence swaps source and destination.
theorem linear_equiv_symm_src[R: Ring, M: AddCommGroup, N: AddCommGroup](
    e: LinearEquiv[R, M, N]
) {
    linear_equiv_symm(e).src = e.dst
} by {
    LinearEquiv.new(e.dst, e.src, e.inv_fn, e.to_fn) = Option.some(linear_equiv_symm(e))
}

/// The inverse linear equivalence swaps destination and source.
theorem linear_equiv_symm_dst[R: Ring, M: AddCommGroup, N: AddCommGroup](
    e: LinearEquiv[R, M, N]
) {
    linear_equiv_symm(e).dst = e.src
} by {
    LinearEquiv.new(e.dst, e.src, e.inv_fn, e.to_fn) = Option.some(linear_equiv_symm(e))
}

/// A bundled linear equivalence built from a module homomorphism together with a
/// two-sided inverse of its underlying function. Defined exactly when the supplied
/// function is a two-sided inverse of the homomorphism's underlying function.
define linear_equiv_of_module_hom[R: Ring, M: AddCommGroup, N: AddCommGroup](
    f: ModuleHom[R, M, N], g: N -> M
) -> Option[LinearEquiv[R, M, N]] {
    LinearEquiv[R, M, N].new(f.src, f.dst, f.hom, g)
}

/// When the supplied function is a two-sided inverse of the homomorphism's underlying
/// function, the bundled linear equivalence is defined.
theorem linear_equiv_of_module_hom_some[R: Ring, M: AddCommGroup, N: AddCommGroup](
    f: ModuleHom[R, M, N], g: N -> M
) {
    is_two_sided_inverse_fn(f.hom, g) implies exists(h: LinearEquiv[R, M, N]) {
        linear_equiv_of_module_hom(f, g) = Option.some(h)
    }
} by {
    if is_two_sided_inverse_fn(f.hom, g) {
        module_hom_is_linear_map(f)
        is_linear_map(f.src, f.dst, f.hom) and is_two_sided_inverse_fn(f.hom, g)
        let h: LinearEquiv[R, M, N] satisfy {
            LinearEquiv[R, M, N].new(f.src, f.dst, f.hom, g) = Option.some(h)
        }
        linear_equiv_of_module_hom(f, g) = Option.some(h)
    }
}

/// The linear equivalence built from a homomorphism has the homomorphism's source as
/// its source.
theorem linear_equiv_of_module_hom_src[R: Ring, M: AddCommGroup, N: AddCommGroup](
    f: ModuleHom[R, M, N], g: N -> M, h: LinearEquiv[R, M, N]
) {
    linear_equiv_of_module_hom(f, g) = Option.some(h) implies h.src = f.src
} by {
    if linear_equiv_of_module_hom(f, g) = Option.some(h) {
        LinearEquiv[R, M, N].new(f.src, f.dst, f.hom, g) = Option.some(h)
        h.src = f.src
    }
}

/// The linear equivalence built from a homomorphism has the homomorphism's destination
/// as its destination.
theorem linear_equiv_of_module_hom_dst[R: Ring, M: AddCommGroup, N: AddCommGroup](
    f: ModuleHom[R, M, N], g: N -> M, h: LinearEquiv[R, M, N]
) {
    linear_equiv_of_module_hom(f, g) = Option.some(h) implies h.dst = f.dst
} by {
    if linear_equiv_of_module_hom(f, g) = Option.some(h) {
        LinearEquiv[R, M, N].new(f.src, f.dst, f.hom, g) = Option.some(h)
        h.dst = f.dst
    }
}

/// The composition of linear equivalences, defined when the source of the first
/// equals the destination of the second.
define linear_equiv_compose[R: Ring, M: AddCommGroup, N: AddCommGroup, K: AddCommGroup](
    e: LinearEquiv[R, N, K], d: LinearEquiv[R, M, N]
) -> Option[LinearEquiv[R, M, K]] {
    LinearEquiv[R, M, K].new(d.src, e.dst, compose(e.to_fn, d.to_fn), compose(d.inv_fn, e.inv_fn))
}

/// When the source of the first equivalence equals the destination of the second, the
/// composition is defined.
theorem linear_equiv_compose_some[R: Ring, M: AddCommGroup, N: AddCommGroup, K: AddCommGroup](
    e: LinearEquiv[R, N, K], d: LinearEquiv[R, M, N]
) {
    e.src = d.dst implies exists(h: LinearEquiv[R, M, K]) {
        linear_equiv_compose(e, d) = Option.some(h)
    }
} by {
    if e.src = d.dst {
        linear_equiv_to_fn_is_linear_map(e)
        linear_equiv_to_fn_is_linear_map(d)
        compose_is_linear_map(d.src, d.dst, e.dst, e.to_fn, d.to_fn)
        linear_equiv_to_inv_two_sided(e)
        linear_equiv_to_inv_two_sided(d)
        compose_two_sided_inverse_fn(d.to_fn, e.to_fn, d.inv_fn, e.inv_fn)
        is_linear_map(d.src, e.dst, compose(e.to_fn, d.to_fn)) and
            is_two_sided_inverse_fn(compose(e.to_fn, d.to_fn), compose(d.inv_fn, e.inv_fn))
        let h: LinearEquiv[R, M, K] satisfy {
            LinearEquiv[R, M, K].new(d.src, e.dst, compose(e.to_fn, d.to_fn), compose(d.inv_fn, e.inv_fn)) = Option.some(h)
        }
        linear_equiv_compose(e, d) = Option.some(h)
    }
}

/// The composed linear equivalence has the source of the second equivalence as its
/// source.
theorem linear_equiv_compose_src[R: Ring, M: AddCommGroup, N: AddCommGroup, K: AddCommGroup](
    e: LinearEquiv[R, N, K], d: LinearEquiv[R, M, N], h: LinearEquiv[R, M, K]
) {
    linear_equiv_compose(e, d) = Option.some(h) implies h.src = d.src
} by {
    if linear_equiv_compose(e, d) = Option.some(h) {
        LinearEquiv[R, M, K].new(d.src, e.dst, compose(e.to_fn, d.to_fn), compose(d.inv_fn, e.inv_fn)) =
            Option.some(h)
        h.src = d.src
    }
}

/// The composed linear equivalence has the destination of the first equivalence as its
/// destination.
theorem linear_equiv_compose_dst[R: Ring, M: AddCommGroup, N: AddCommGroup, K: AddCommGroup](
    e: LinearEquiv[R, N, K], d: LinearEquiv[R, M, N], h: LinearEquiv[R, M, K]
) {
    linear_equiv_compose(e, d) = Option.some(h) implies h.dst = e.dst
} by {
    if linear_equiv_compose(e, d) = Option.some(h) {
        LinearEquiv[R, M, K].new(d.src, e.dst, compose(e.to_fn, d.to_fn), compose(d.inv_fn, e.inv_fn)) =
            Option.some(h)
        h.dst = e.dst
    }
}

/// The inverse function of a linear equivalence is a linear equivalence in the
/// predicate sense.
theorem linear_equiv_inv_fn_is_linear_equiv[R: Ring, M: AddCommGroup, N: AddCommGroup](
    e: LinearEquiv[R, M, N]
) {
    is_linear_equiv(e.dst, e.src, e.inv_fn)
} by {
    linear_equiv_to_fn_is_linear_equiv(e)
    linear_equiv_to_inv_two_sided(e)
    linear_equiv_two_sided_inverse_is_linear_equiv(e.src, e.dst, e.to_fn, e.inv_fn)
}

/// The forward function of a linear equivalence is injective.
theorem linear_equiv_to_fn_is_injective[R: Ring, M: AddCommGroup, N: AddCommGroup](
    e: LinearEquiv[R, M, N]
) {
    is_injective_fn(e.to_fn)
} by {
    linear_equiv_to_fn_is_linear_equiv(e)
    linear_equiv_is_injective(e.src, e.dst, e.to_fn)
}

/// The forward function of a linear equivalence is surjective.
theorem linear_equiv_to_fn_is_surjective[R: Ring, M: AddCommGroup, N: AddCommGroup](
    e: LinearEquiv[R, M, N]
) {
    is_surjective_fn(e.to_fn)
} by {
    linear_equiv_to_fn_is_linear_equiv(e)
    linear_equiv_is_surjective(e.src, e.dst, e.to_fn)
}

/// The inverse function of a linear equivalence sends zero to zero.
theorem linear_equiv_inv_map_zero[R: Ring, M: AddCommGroup, N: AddCommGroup](
    e: LinearEquiv[R, M, N]
) {
    e.inv_fn(N.0) = M.0
} by {
    linear_equiv_to_fn_is_linear_equiv(e)
    linear_equiv_to_inv_two_sided(e)
    linear_equiv_two_sided_inverse_zero(e.src, e.dst, e.to_fn, e.inv_fn)
}

/// The inverse function of a linear equivalence preserves addition.
theorem linear_equiv_inv_map_add[R: Ring, M: AddCommGroup, N: AddCommGroup](
    e: LinearEquiv[R, M, N], x: N, y: N
) {
    e.inv_fn(x + y) = e.inv_fn(x) + e.inv_fn(y)
} by {
    linear_equiv_to_fn_is_linear_equiv(e)
    linear_equiv_to_inv_two_sided(e)
    linear_equiv_two_sided_inverse_add(e.src, e.dst, e.to_fn, e.inv_fn, x, y)
}

/// The inverse function of a linear equivalence preserves the scalar action.
theorem linear_equiv_inv_map_smul[R: Ring, M: AddCommGroup, N: AddCommGroup](
    e: LinearEquiv[R, M, N], r: R, x: N
) {
    e.inv_fn(e.dst.smul(r, x)) = e.src.smul(r, e.inv_fn(x))
} by {
    linear_equiv_to_fn_is_linear_equiv(e)
    linear_equiv_to_inv_two_sided(e)
    linear_equiv_two_sided_inverse_smul(e.src, e.dst, e.to_fn, e.inv_fn, r, x)
}

/// The inverse function of a linear equivalence preserves negation.
theorem linear_equiv_inv_map_neg[R: Ring, M: AddCommGroup, N: AddCommGroup](
    e: LinearEquiv[R, M, N], x: N
) {
    e.inv_fn(-x) = -e.inv_fn(x)
} by {
    linear_equiv_to_fn_is_linear_equiv(e)
    linear_equiv_to_inv_two_sided(e)
    linear_equiv_two_sided_inverse_neg(e.src, e.dst, e.to_fn, e.inv_fn, x)
}

/// The inverse function of a linear equivalence preserves subtraction.
theorem linear_equiv_inv_map_sub[R: Ring, M: AddCommGroup, N: AddCommGroup](
    e: LinearEquiv[R, M, N], x: N, y: N
) {
    e.inv_fn(x - y) = e.inv_fn(x) - e.inv_fn(y)
} by {
    linear_equiv_to_fn_is_linear_equiv(e)
    linear_equiv_to_inv_two_sided(e)
    linear_equiv_two_sided_inverse_sub(e.src, e.dst, e.to_fn, e.inv_fn, x, y)
}

/// A point maps forward to a target exactly when the inverse of the target is that
/// point.
theorem linear_equiv_to_fn_apply_eq_iff[R: Ring, M: AddCommGroup, N: AddCommGroup](
    e: LinearEquiv[R, M, N], x: M, y: N
) {
    e.to_fn(x) = y iff x = e.inv_fn(y)
} by {
    linear_equiv_to_fn_is_linear_equiv(e)
    linear_equiv_to_inv_two_sided(e)
    linear_equiv_apply_eq_iff(e.src, e.dst, e.to_fn, e.inv_fn, x, y)
}

/// A point maps forward to zero exactly when it is zero.
theorem linear_equiv_map_eq_zero_iff_eq_zero[R: Ring, M: AddCommGroup, N: AddCommGroup](
    e: LinearEquiv[R, M, N], x: M
) {
    e.to_fn(x) = N.0 iff x = M.0
} by {
    linear_equiv_to_fn_is_linear_equiv(e)
    linear_equiv_apply_eq_zero_iff_eq_zero(e.src, e.dst, e.to_fn, x)
}

/// The forward function of a linear equivalence agrees on two points exactly when the
/// points are equal.
theorem linear_equiv_to_fn_eq_iff[R: Ring, M: AddCommGroup, N: AddCommGroup](
    e: LinearEquiv[R, M, N], x: M, y: M
) {
    e.to_fn(x) = e.to_fn(y) iff x = y
} by {
    linear_equiv_to_fn_is_injective(e)
    if e.to_fn(x) = e.to_fn(y) {
        is_injective_fn(e.to_fn)
        x = y
    }
}

/// The inverse function of a linear equivalence agrees on two points exactly when the
/// points are equal.
theorem linear_equiv_inv_fn_eq_iff[R: Ring, M: AddCommGroup, N: AddCommGroup](
    e: LinearEquiv[R, M, N], x: N, y: N
) {
    e.inv_fn(x) = e.inv_fn(y) iff x = y
} by {
    if e.inv_fn(x) = e.inv_fn(y) {
        linear_equiv_right_inverse(e, x)
        linear_equiv_right_inverse(e, y)
        x = y
    }
}

/// The inverse of a target equals a point exactly when the point maps forward to the
/// target.
theorem linear_equiv_inv_apply_eq_iff[R: Ring, M: AddCommGroup, N: AddCommGroup](
    e: LinearEquiv[R, M, N], x: M, y: N
) {
    e.inv_fn(y) = x iff y = e.to_fn(x)
} by {
    linear_equiv_to_fn_apply_eq_iff(e, x, y)
}

/// The inverse function maps a target to zero exactly when the target is zero.
theorem linear_equiv_inv_map_eq_zero_iff_eq_zero[R: Ring, M: AddCommGroup, N: AddCommGroup](
    e: LinearEquiv[R, M, N], y: N
) {
    e.inv_fn(y) = M.0 iff y = N.0
} by {
    linear_equiv_inv_fn_is_linear_equiv(e)
    linear_equiv_apply_eq_zero_iff_eq_zero(e.dst, e.src, e.inv_fn, y)
}

/// The identity linear equivalence witnesses that a module is linearly equivalent to
/// itself.
theorem linear_equiv_identity_modules_linearly_equivalent[R: Ring, M: AddCommGroup](
    m: Module[R, M]
) {
    modules_linearly_equivalent(m, m)
} by {
    linear_equiv_modules_linearly_equivalent(linear_equiv_identity(m))
    linear_equiv_identity_src(m)
    linear_equiv_identity_dst(m)
}

/// The inverse linear equivalence witnesses that the destination is linearly equivalent
/// to the source.
theorem linear_equiv_symm_modules_linearly_equivalent[R: Ring, M: AddCommGroup, N: AddCommGroup](
    e: LinearEquiv[R, M, N]
) {
    modules_linearly_equivalent(e.dst, e.src)
} by {
    linear_equiv_modules_linearly_equivalent(linear_equiv_symm(e))
    linear_equiv_symm_src(e)
    linear_equiv_symm_dst(e)
}

/// A linear equivalence built from a module homomorphism and a two-sided inverse of its
/// underlying function witnesses that the source and destination are linearly equivalent.
theorem linear_equiv_of_module_hom_modules_linearly_equivalent[R: Ring, M: AddCommGroup, N: AddCommGroup](
    f: ModuleHom[R, M, N], g: N -> M
) {
    is_two_sided_inverse_fn(f.hom, g) implies modules_linearly_equivalent(f.src, f.dst)
} by {
    if is_two_sided_inverse_fn(f.hom, g) {
        linear_equiv_of_module_hom_some(f, g)
        let h: LinearEquiv[R, M, N] satisfy {
            linear_equiv_of_module_hom(f, g) = Option.some(h)
        }
        linear_equiv_of_module_hom_src(f, g, h)
        linear_equiv_of_module_hom_dst(f, g, h)
        linear_equiv_modules_linearly_equivalent(h)
        modules_linearly_equivalent(f.src, f.dst)
    }
}

/// The inverse function of a linear equivalence is injective.
theorem linear_equiv_inv_fn_is_injective[R: Ring, M: AddCommGroup, N: AddCommGroup](
    e: LinearEquiv[R, M, N]
) {
    is_injective_fn(e.inv_fn)
} by {
    linear_equiv_inv_fn_is_linear_equiv(e)
    linear_equiv_is_injective(e.dst, e.src, e.inv_fn)
}

/// The inverse function of a linear equivalence is surjective.
theorem linear_equiv_inv_fn_is_surjective[R: Ring, M: AddCommGroup, N: AddCommGroup](
    e: LinearEquiv[R, M, N]
) {
    is_surjective_fn(e.inv_fn)
} by {
    linear_equiv_inv_fn_is_linear_equiv(e)
    linear_equiv_is_surjective(e.dst, e.src, e.inv_fn)
}

/// The forward function of a linear equivalence is a bijection.
theorem linear_equiv_to_fn_is_bijection[R: Ring, M: AddCommGroup, N: AddCommGroup](
    e: LinearEquiv[R, M, N]
) {
    is_bijection_fn(e.to_fn)
} by {
    linear_equiv_to_fn_is_linear_equiv(e)
    linear_equiv_is_bijection(e.src, e.dst, e.to_fn)
}

/// The inverse function of a linear equivalence is a bijection.
theorem linear_equiv_inv_fn_is_bijection[R: Ring, M: AddCommGroup, N: AddCommGroup](
    e: LinearEquiv[R, M, N]
) {
    is_bijection_fn(e.inv_fn)
} by {
    linear_equiv_inv_fn_is_linear_equiv(e)
    linear_equiv_is_bijection(e.dst, e.src, e.inv_fn)
}

/// The forward function of a linear equivalence disagrees on two points exactly when the
/// points are unequal.
theorem linear_equiv_to_fn_ne_iff[R: Ring, M: AddCommGroup, N: AddCommGroup](
    e: LinearEquiv[R, M, N], x: M, y: M
) {
    e.to_fn(x) != e.to_fn(y) iff x != y
} by {
    linear_equiv_to_fn_eq_iff(e, x, y)
}

/// The inverse function of a linear equivalence disagrees on two points exactly when the
/// points are unequal.
theorem linear_equiv_inv_fn_ne_iff[R: Ring, M: AddCommGroup, N: AddCommGroup](
    e: LinearEquiv[R, M, N], x: N, y: N
) {
    e.inv_fn(x) != e.inv_fn(y) iff x != y
} by {
    linear_equiv_inv_fn_eq_iff(e, x, y)
}

/// A point maps forward to a nonzero target exactly when it is nonzero.
theorem linear_equiv_map_ne_zero_iff[R: Ring, M: AddCommGroup, N: AddCommGroup](
    e: LinearEquiv[R, M, N], x: M
) {
    e.to_fn(x) != N.0 iff x != M.0
} by {
    linear_equiv_map_eq_zero_iff_eq_zero(e, x)
}

/// The inverse function maps a target to a nonzero point exactly when the target is
/// nonzero.
theorem linear_equiv_inv_map_ne_zero_iff[R: Ring, M: AddCommGroup, N: AddCommGroup](
    e: LinearEquiv[R, M, N], y: N
) {
    e.inv_fn(y) != M.0 iff y != N.0
} by {
    linear_equiv_inv_map_eq_zero_iff_eq_zero(e, y)
}

/// A composed linear equivalence witnesses that the source of the second equivalence is
/// linearly equivalent to the destination of the first.
theorem linear_equiv_compose_modules_linearly_equivalent[R: Ring, M: AddCommGroup, N: AddCommGroup, K: AddCommGroup](
    e: LinearEquiv[R, N, K], d: LinearEquiv[R, M, N]
) {
    e.src = d.dst implies modules_linearly_equivalent(d.src, e.dst)
} by {
    if e.src = d.dst {
        linear_equiv_compose_some(e, d)
        let h: LinearEquiv[R, M, K] satisfy {
            linear_equiv_compose(e, d) = Option.some(h)
        }
        linear_equiv_compose_src(e, d, h)
        linear_equiv_compose_dst(e, d, h)
        linear_equiv_modules_linearly_equivalent(h)
        modules_linearly_equivalent(d.src, e.dst)
    }
}

/// A point maps forward to a target distinct from a given value exactly when it is
/// distinct from the inverse image of that value.
theorem linear_equiv_to_fn_apply_ne_iff[R: Ring, M: AddCommGroup, N: AddCommGroup](
    e: LinearEquiv[R, M, N], x: M, y: N
) {
    e.to_fn(x) != y iff x != e.inv_fn(y)
} by {
    linear_equiv_to_fn_apply_eq_iff(e, x, y)
}

/// The inverse of a target differs from a given point exactly when the target differs
/// from the forward image of that point.
theorem linear_equiv_inv_apply_ne_iff[R: Ring, M: AddCommGroup, N: AddCommGroup](
    e: LinearEquiv[R, M, N], x: M, y: N
) {
    e.inv_fn(y) != x iff y != e.to_fn(x)
} by {
    linear_equiv_inv_apply_eq_iff(e, x, y)
}

/// Composing a linear equivalence with its inverse on the left is defined.
theorem linear_equiv_compose_symm_left_some[R: Ring, M: AddCommGroup, N: AddCommGroup](
    e: LinearEquiv[R, M, N]
) {
    exists(h: LinearEquiv[R, M, M]) {
        linear_equiv_compose(linear_equiv_symm(e), e) = Option.some(h)
    }
} by {
    linear_equiv_symm_src(e)
    linear_equiv_compose_some(linear_equiv_symm(e), e)
}

/// Composing a linear equivalence with its inverse on the right is defined.
theorem linear_equiv_compose_symm_right_some[R: Ring, M: AddCommGroup, N: AddCommGroup](
    e: LinearEquiv[R, M, N]
) {
    exists(h: LinearEquiv[R, N, N]) {
        linear_equiv_compose(e, linear_equiv_symm(e)) = Option.some(h)
    }
} by {
    linear_equiv_symm_dst(e)
    linear_equiv_compose_some(e, linear_equiv_symm(e))
}

/// The composition of a linear equivalence with its inverse on the left has source
/// equal to the destination of the original equivalence.
theorem linear_equiv_compose_symm_left_src[R: Ring, M: AddCommGroup, N: AddCommGroup](
    e: LinearEquiv[R, M, N], h: LinearEquiv[R, M, M]
) {
    linear_equiv_compose(linear_equiv_symm(e), e) = Option.some(h)
    implies h.src = e.src
} by {
    if linear_equiv_compose(linear_equiv_symm(e), e) = Option.some(h) {
        linear_equiv_symm_src(e)
        linear_equiv_symm(e).src = e.dst
        linear_equiv_compose_src(linear_equiv_symm(e), e, h)
    }
}

/// The composition of a linear equivalence with its inverse on the left has destination
/// equal to the source of the original equivalence.
theorem linear_equiv_compose_symm_left_dst[R: Ring, M: AddCommGroup, N: AddCommGroup](
    e: LinearEquiv[R, M, N], h: LinearEquiv[R, M, M]
) {
    linear_equiv_compose(linear_equiv_symm(e), e) = Option.some(h)
    implies h.dst = e.src
} by {
    if linear_equiv_compose(linear_equiv_symm(e), e) = Option.some(h) {
        linear_equiv_symm_src(e)
        linear_equiv_symm(e).src = e.dst
        linear_equiv_compose_dst(linear_equiv_symm(e), e, h)
        linear_equiv_symm_dst(e)
    }
}

/// The composition of a linear equivalence with its inverse on the right has source
/// equal to the destination of the original equivalence.
theorem linear_equiv_compose_symm_right_src[R: Ring, M: AddCommGroup, N: AddCommGroup](
    e: LinearEquiv[R, M, N], h: LinearEquiv[R, N, N]
) {
    linear_equiv_compose(e, linear_equiv_symm(e)) = Option.some(h)
    implies h.src = e.dst
} by {
    if linear_equiv_compose(e, linear_equiv_symm(e)) = Option.some(h) {
        linear_equiv_symm_dst(e)
        linear_equiv_symm(e).dst = e.src
        linear_equiv_compose_src(e, linear_equiv_symm(e), h)
        linear_equiv_symm_src(e)
    }
}

/// The composition of a linear equivalence with its inverse on the right has destination
/// equal to the destination of the original equivalence.
theorem linear_equiv_compose_symm_right_dst[R: Ring, M: AddCommGroup, N: AddCommGroup](
    e: LinearEquiv[R, M, N], h: LinearEquiv[R, N, N]
) {
    linear_equiv_compose(e, linear_equiv_symm(e)) = Option.some(h)
    implies h.dst = e.dst
} by {
    if linear_equiv_compose(e, linear_equiv_symm(e)) = Option.some(h) {
        linear_equiv_symm_dst(e)
        linear_equiv_symm(e).dst = e.src
        linear_equiv_compose_dst(e, linear_equiv_symm(e), h)
    }
}

/// Composing a linear equivalence with the identity equivalence on its destination
/// (on the left) is defined.
theorem linear_equiv_compose_identity_left_some[R: Ring, M: AddCommGroup, N: AddCommGroup](
    e: LinearEquiv[R, M, N]
) {
    exists(h: LinearEquiv[R, M, N]) {
        linear_equiv_compose(linear_equiv_identity(e.dst), e) = Option.some(h)
    }
} by {
    linear_equiv_identity_src(e.dst)
    linear_equiv_compose_some(linear_equiv_identity(e.dst), e)
}

/// Composing a linear equivalence with the identity equivalence on its source
/// (on the right) is defined.
theorem linear_equiv_compose_identity_right_some[R: Ring, M: AddCommGroup, N: AddCommGroup](
    e: LinearEquiv[R, M, N]
) {
    exists(h: LinearEquiv[R, M, N]) {
        linear_equiv_compose(e, linear_equiv_identity(e.src)) = Option.some(h)
    }
} by {
    linear_equiv_identity_dst(e.src)
    linear_equiv_compose_some(e, linear_equiv_identity(e.src))
}

/// The composition of a linear equivalence with the identity on its destination has
/// the same source as the original equivalence.
theorem linear_equiv_compose_identity_left_src[R: Ring, M: AddCommGroup, N: AddCommGroup](
    e: LinearEquiv[R, M, N], h: LinearEquiv[R, M, N]
) {
    linear_equiv_compose(linear_equiv_identity(e.dst), e) = Option.some(h)
    implies h.src = e.src
} by {
    if linear_equiv_compose(linear_equiv_identity(e.dst), e) = Option.some(h) {
        linear_equiv_identity_src(e.dst)
        linear_equiv_identity(e.dst).src = e.dst
        linear_equiv_compose_src(linear_equiv_identity(e.dst), e, h)
    }
}

/// The composition of a linear equivalence with the identity on its destination has
/// the same destination as the original equivalence.
theorem linear_equiv_compose_identity_left_dst[R: Ring, M: AddCommGroup, N: AddCommGroup](
    e: LinearEquiv[R, M, N], h: LinearEquiv[R, M, N]
) {
    linear_equiv_compose(linear_equiv_identity(e.dst), e) = Option.some(h)
    implies h.dst = e.dst
} by {
    if linear_equiv_compose(linear_equiv_identity(e.dst), e) = Option.some(h) {
        linear_equiv_identity_src(e.dst)
        linear_equiv_identity(e.dst).src = e.dst
        linear_equiv_compose_dst(linear_equiv_identity(e.dst), e, h)
        linear_equiv_identity_dst(e.dst)
    }
}

/// The composition of a linear equivalence with the identity on its source has the
/// same source as the original equivalence.
theorem linear_equiv_compose_identity_right_src[R: Ring, M: AddCommGroup, N: AddCommGroup](
    e: LinearEquiv[R, M, N], h: LinearEquiv[R, M, N]
) {
    linear_equiv_compose(e, linear_equiv_identity(e.src)) = Option.some(h)
    implies h.src = e.src
} by {
    if linear_equiv_compose(e, linear_equiv_identity(e.src)) = Option.some(h) {
        linear_equiv_identity_dst(e.src)
        linear_equiv_identity(e.src).dst = e.src
        linear_equiv_compose_src(e, linear_equiv_identity(e.src), h)
        linear_equiv_identity_src(e.src)
    }
}

/// The composition of a linear equivalence with the identity on its source has the
/// same destination as the original equivalence.
theorem linear_equiv_compose_identity_right_dst[R: Ring, M: AddCommGroup, N: AddCommGroup](
    e: LinearEquiv[R, M, N], h: LinearEquiv[R, M, N]
) {
    linear_equiv_compose(e, linear_equiv_identity(e.src)) = Option.some(h)
    implies h.dst = e.dst
} by {
    if linear_equiv_compose(e, linear_equiv_identity(e.src)) = Option.some(h) {
        linear_equiv_identity_dst(e.src)
        linear_equiv_identity(e.src).dst = e.src
        linear_equiv_compose_dst(e, linear_equiv_identity(e.src), h)
    }
}

/// Composing an equivalence with its inverse on the left witnesses that the source
/// is linearly equivalent to itself.
theorem linear_equiv_compose_symm_left_modules_linearly_equivalent[R: Ring, M: AddCommGroup, N: AddCommGroup](
    e: LinearEquiv[R, M, N], h: LinearEquiv[R, M, M]
) {
    linear_equiv_compose(linear_equiv_symm(e), e) = Option.some(h)
        implies modules_linearly_equivalent(e.src, e.src)
} by {
    if linear_equiv_compose(linear_equiv_symm(e), e) = Option.some(h) {
        linear_equiv_compose_symm_left_src(e, h)
        linear_equiv_compose_symm_left_dst(e, h)
        linear_equiv_modules_linearly_equivalent(h)
    }
}

/// Composing an equivalence with its inverse on the right witnesses that the destination
/// is linearly equivalent to itself.
theorem linear_equiv_compose_symm_right_modules_linearly_equivalent[R: Ring, M: AddCommGroup, N: AddCommGroup](
    e: LinearEquiv[R, M, N], h: LinearEquiv[R, N, N]
) {
    linear_equiv_compose(e, linear_equiv_symm(e)) = Option.some(h)
        implies modules_linearly_equivalent(e.dst, e.dst)
} by {
    if linear_equiv_compose(e, linear_equiv_symm(e)) = Option.some(h) {
        linear_equiv_compose_symm_right_src(e, h)
        linear_equiv_compose_symm_right_dst(e, h)
        linear_equiv_modules_linearly_equivalent(h)
    }
}

/// Composing an equivalence with the identity on its destination witnesses that the
/// source and destination are linearly equivalent.
theorem linear_equiv_compose_identity_left_modules_linearly_equivalent[R: Ring, M: AddCommGroup, N: AddCommGroup](
    e: LinearEquiv[R, M, N], h: LinearEquiv[R, M, N]
) {
    linear_equiv_compose(linear_equiv_identity(e.dst), e) = Option.some(h)
        implies modules_linearly_equivalent(e.src, e.dst)
} by {
    if linear_equiv_compose(linear_equiv_identity(e.dst), e) = Option.some(h) {
        linear_equiv_compose_identity_left_src(e, h)
        linear_equiv_compose_identity_left_dst(e, h)
        linear_equiv_modules_linearly_equivalent(h)
    }
}

/// Composing an equivalence with the identity on its source witnesses that the source
/// and destination are linearly equivalent.
theorem linear_equiv_compose_identity_right_modules_linearly_equivalent[R: Ring, M: AddCommGroup, N: AddCommGroup](
    e: LinearEquiv[R, M, N], h: LinearEquiv[R, M, N]
) {
    linear_equiv_compose(e, linear_equiv_identity(e.src)) = Option.some(h)
        implies modules_linearly_equivalent(e.src, e.dst)
} by {
    if linear_equiv_compose(e, linear_equiv_identity(e.src)) = Option.some(h) {
        linear_equiv_compose_identity_right_src(e, h)
        linear_equiv_compose_identity_right_dst(e, h)
        linear_equiv_modules_linearly_equivalent(h)
    }
}

/// The double inverse of a linear equivalence has the same source as the original.
theorem linear_equiv_symm_symm_src[R: Ring, M: AddCommGroup, N: AddCommGroup](
    e: LinearEquiv[R, M, N]
) {
    linear_equiv_symm(linear_equiv_symm(e)).src = e.src
} by {
    linear_equiv_symm_src(linear_equiv_symm(e))
    linear_equiv_symm_dst(e)
}

/// The double inverse of a linear equivalence has the same destination as the original.
theorem linear_equiv_symm_symm_dst[R: Ring, M: AddCommGroup, N: AddCommGroup](
    e: LinearEquiv[R, M, N]
) {
    linear_equiv_symm(linear_equiv_symm(e)).dst = e.dst
} by {
    linear_equiv_symm_dst(linear_equiv_symm(e))
    linear_equiv_symm_src(e)
}

/// The double inverse of a linear equivalence witnesses that the original source and
/// destination are linearly equivalent.
theorem linear_equiv_symm_symm_modules_linearly_equivalent[R: Ring, M: AddCommGroup, N: AddCommGroup](
    e: LinearEquiv[R, M, N]
) {
    modules_linearly_equivalent(e.src, e.dst)
} by {
    linear_equiv_symm_symm_src(e)
    linear_equiv_symm_symm_dst(e)
    linear_equiv_modules_linearly_equivalent(linear_equiv_symm(linear_equiv_symm(e)))
}

/// A bundled linear equivalence witnesses that its destination and source are linearly
/// equivalent (the symmetric form).
theorem linear_equiv_modules_linearly_equivalent_symm[R: Ring, M: AddCommGroup, N: AddCommGroup](
    e: LinearEquiv[R, M, N]
) {
    modules_linearly_equivalent(e.dst, e.src)
} by {
    linear_equiv_modules_linearly_equivalent(e)
    modules_linearly_equivalent_symm(e.src, e.dst)
}

/// Reflexivity at the source of a bundled linear equivalence.
theorem linear_equiv_modules_linearly_equivalent_refl_src[R: Ring, M: AddCommGroup, N: AddCommGroup](
    e: LinearEquiv[R, M, N]
) {
    modules_linearly_equivalent(e.src, e.src)
} by {
    modules_linearly_equivalent_refl(e.src)
}

/// Reflexivity at the destination of a bundled linear equivalence.
theorem linear_equiv_modules_linearly_equivalent_refl_dst[R: Ring, M: AddCommGroup, N: AddCommGroup](
    e: LinearEquiv[R, M, N]
) {
    modules_linearly_equivalent(e.dst, e.dst)
} by {
    modules_linearly_equivalent_refl(e.dst)
}

/// Two bundled linear equivalences whose middle endpoints agree witness that the
/// outer source and destination are linearly equivalent (transitivity bridge).
theorem linear_equiv_trans_modules_linearly_equivalent[R: Ring, M: AddCommGroup, N: AddCommGroup, K: AddCommGroup](
    e: LinearEquiv[R, M, N], d: LinearEquiv[R, N, K]
) {
    e.dst = d.src implies modules_linearly_equivalent(e.src, d.dst)
} by {
    if e.dst = d.src {
        linear_equiv_modules_linearly_equivalent(e)
        linear_equiv_modules_linearly_equivalent(d)
        modules_linearly_equivalent_trans(e.src, e.dst, d.dst)
    }
}

/// Linear equivalence of modules is symmetric, in biconditional form.
theorem modules_linearly_equivalent_symm_iff[R: Ring, M: AddCommGroup, N: AddCommGroup](
    src: Module[R, M], dst: Module[R, N]) {
    modules_linearly_equivalent(src, dst) iff modules_linearly_equivalent(dst, src)
} by {
    modules_linearly_equivalent_symm(src, dst)
    modules_linearly_equivalent_symm(dst, src)
}

/// The inverse of the identity linear equivalence on a module has source equal to
/// the module.
theorem linear_equiv_identity_symm_src[R: Ring, M: AddCommGroup](m: Module[R, M]) {
    linear_equiv_symm(linear_equiv_identity(m)).src = m
} by {
    linear_equiv_symm_src(linear_equiv_identity(m))
    linear_equiv_identity_dst(m)
}

/// The inverse of the identity linear equivalence on a module has destination equal
/// to the module.
theorem linear_equiv_identity_symm_dst[R: Ring, M: AddCommGroup](m: Module[R, M]) {
    linear_equiv_symm(linear_equiv_identity(m)).dst = m
} by {
    linear_equiv_symm_dst(linear_equiv_identity(m))
    linear_equiv_identity_src(m)
}

/// The inverse of the identity linear equivalence witnesses that the module is
/// linearly equivalent to itself.
theorem linear_equiv_identity_symm_modules_linearly_equivalent[R: Ring, M: AddCommGroup](
    m: Module[R, M]
) {
    modules_linearly_equivalent(m, m)
} by {
    linear_equiv_identity_symm_src(m)
    linear_equiv_identity_symm_dst(m)
    linear_equiv_modules_linearly_equivalent(linear_equiv_symm(linear_equiv_identity(m)))
}

/// The double inverse of a bundled linear equivalence witnesses linear equivalence
/// between the source and source (reflexivity at the source via double symmetry).
theorem linear_equiv_symm_symm_modules_linearly_equivalent_refl_src[R: Ring, M: AddCommGroup, N: AddCommGroup](
    e: LinearEquiv[R, M, N]
) {
    modules_linearly_equivalent(linear_equiv_symm(linear_equiv_symm(e)).src, e.src)
} by {
    linear_equiv_symm_symm_src(e)
    modules_linearly_equivalent_refl(e.src)
}

/// The double inverse of a bundled linear equivalence witnesses linear equivalence
/// between the destination and destination (reflexivity at the destination via double
/// symmetry).
theorem linear_equiv_symm_symm_modules_linearly_equivalent_refl_dst[R: Ring, M: AddCommGroup, N: AddCommGroup](
    e: LinearEquiv[R, M, N]
) {
    modules_linearly_equivalent(linear_equiv_symm(linear_equiv_symm(e)).dst, e.dst)
} by {
    linear_equiv_symm_symm_dst(e)
    modules_linearly_equivalent_refl(e.dst)
}

/// The inverse of the identity linear equivalence witnesses linear equivalence between
/// its source and the original module (reflexivity bridge at the source).
theorem linear_equiv_identity_symm_modules_linearly_equivalent_refl_src[R: Ring, M: AddCommGroup](
    m: Module[R, M]
) {
    modules_linearly_equivalent(linear_equiv_symm(linear_equiv_identity(m)).src, m)
} by {
    linear_equiv_identity_symm_src(m)
    modules_linearly_equivalent_refl(m)
}

/// The inverse of the identity linear equivalence witnesses linear equivalence between
/// its destination and the original module (reflexivity bridge at the destination).
theorem linear_equiv_identity_symm_modules_linearly_equivalent_refl_dst[R: Ring, M: AddCommGroup](
    m: Module[R, M]
) {
    modules_linearly_equivalent(linear_equiv_symm(linear_equiv_identity(m)).dst, m)
} by {
    linear_equiv_identity_symm_dst(m)
    modules_linearly_equivalent_refl(m)
}

/// The source of the inverse of a bundled linear equivalence agrees with the original's
/// destination via reflexivity of linear equivalence.
theorem linear_equiv_symm_modules_linearly_equivalent_refl_src[R: Ring, M: AddCommGroup, N: AddCommGroup](
    e: LinearEquiv[R, M, N]
) {
    modules_linearly_equivalent(linear_equiv_symm(e).src, e.dst)
} by {
    linear_equiv_symm_src(e)
    modules_linearly_equivalent_refl(e.dst)
}

/// The destination of the inverse of a bundled linear equivalence agrees with the
/// original's source via reflexivity of linear equivalence.
theorem linear_equiv_symm_modules_linearly_equivalent_refl_dst[R: Ring, M: AddCommGroup, N: AddCommGroup](
    e: LinearEquiv[R, M, N]
) {
    modules_linearly_equivalent(linear_equiv_symm(e).dst, e.src)
} by {
    linear_equiv_symm_dst(e)
    modules_linearly_equivalent_refl(e.src)
}

/// The source of a composition of a linear equivalence with its inverse on the left agrees
/// with the source of the original equivalence via reflexivity of linear equivalence.
theorem linear_equiv_compose_symm_left_modules_linearly_equivalent_refl_src[R: Ring, M: AddCommGroup, N: AddCommGroup](
    e: LinearEquiv[R, M, N], h: LinearEquiv[R, M, M]
) {
    linear_equiv_compose(linear_equiv_symm(e), e) = Option.some(h)
        implies modules_linearly_equivalent(h.src, e.src)
} by {
    if linear_equiv_compose(linear_equiv_symm(e), e) = Option.some(h) {
        linear_equiv_compose_symm_left_src(e, h)
        modules_linearly_equivalent_refl(e.src)
        modules_linearly_equivalent(h.src, e.src)
    }
}

/// The destination of a composition of a linear equivalence with its inverse on the left
/// agrees with the source of the original equivalence via reflexivity of linear equivalence.
theorem linear_equiv_compose_symm_left_modules_linearly_equivalent_refl_dst[R: Ring, M: AddCommGroup, N: AddCommGroup](
    e: LinearEquiv[R, M, N], h: LinearEquiv[R, M, M]
) {
    linear_equiv_compose(linear_equiv_symm(e), e) = Option.some(h)
        implies modules_linearly_equivalent(h.dst, e.src)
} by {
    if linear_equiv_compose(linear_equiv_symm(e), e) = Option.some(h) {
        linear_equiv_compose_symm_left_dst(e, h)
        modules_linearly_equivalent_refl(e.src)
        modules_linearly_equivalent(h.dst, e.src)
    }
}

/// The source of a composition of a linear equivalence with its inverse on the right agrees
/// with the destination of the original equivalence via reflexivity of linear equivalence.
theorem linear_equiv_compose_symm_right_modules_linearly_equivalent_refl_src[R: Ring, M: AddCommGroup, N: AddCommGroup](
    e: LinearEquiv[R, M, N], h: LinearEquiv[R, N, N]
) {
    linear_equiv_compose(e, linear_equiv_symm(e)) = Option.some(h)
        implies modules_linearly_equivalent(h.src, e.dst)
} by {
    if linear_equiv_compose(e, linear_equiv_symm(e)) = Option.some(h) {
        linear_equiv_compose_symm_right_src(e, h)
        modules_linearly_equivalent_refl(e.dst)
        modules_linearly_equivalent(h.src, e.dst)
    }
}

/// The destination of a composition of a linear equivalence with its inverse on the right
/// agrees with the destination of the original equivalence via reflexivity of linear
/// equivalence.
theorem linear_equiv_compose_symm_right_modules_linearly_equivalent_refl_dst[R: Ring, M: AddCommGroup, N: AddCommGroup](
    e: LinearEquiv[R, M, N], h: LinearEquiv[R, N, N]
) {
    linear_equiv_compose(e, linear_equiv_symm(e)) = Option.some(h)
        implies modules_linearly_equivalent(h.dst, e.dst)
} by {
    if linear_equiv_compose(e, linear_equiv_symm(e)) = Option.some(h) {
        linear_equiv_compose_symm_right_dst(e, h)
        modules_linearly_equivalent_refl(e.dst)
        modules_linearly_equivalent(h.dst, e.dst)
    }
}

/// The source of a composition of a linear equivalence with the identity on its destination
/// agrees with the original's source via reflexivity of linear equivalence.
theorem linear_equiv_compose_identity_left_modules_linearly_equivalent_refl_src[R: Ring, M: AddCommGroup, N: AddCommGroup](
    e: LinearEquiv[R, M, N], h: LinearEquiv[R, M, N]
) {
    linear_equiv_compose(linear_equiv_identity(e.dst), e) = Option.some(h)
        implies modules_linearly_equivalent(h.src, e.src)
} by {
    if linear_equiv_compose(linear_equiv_identity(e.dst), e) = Option.some(h) {
        linear_equiv_compose_identity_left_src(e, h)
        modules_linearly_equivalent_refl(e.src)
        modules_linearly_equivalent(h.src, e.src)
    }
}

/// The destination of a composition of a linear equivalence with the identity on its
/// destination agrees with the original's destination via reflexivity of linear equivalence.
theorem linear_equiv_compose_identity_left_modules_linearly_equivalent_refl_dst[R: Ring, M: AddCommGroup, N: AddCommGroup](
    e: LinearEquiv[R, M, N], h: LinearEquiv[R, M, N]
) {
    linear_equiv_compose(linear_equiv_identity(e.dst), e) = Option.some(h)
        implies modules_linearly_equivalent(h.dst, e.dst)
} by {
    if linear_equiv_compose(linear_equiv_identity(e.dst), e) = Option.some(h) {
        linear_equiv_compose_identity_left_dst(e, h)
        modules_linearly_equivalent_refl(e.dst)
        modules_linearly_equivalent(h.dst, e.dst)
    }
}

/// The source of a composition of a linear equivalence with the identity on its source
/// agrees with the original's source via reflexivity of linear equivalence.
theorem linear_equiv_compose_identity_right_modules_linearly_equivalent_refl_src[R: Ring, M: AddCommGroup, N: AddCommGroup](
    e: LinearEquiv[R, M, N], h: LinearEquiv[R, M, N]
) {
    linear_equiv_compose(e, linear_equiv_identity(e.src)) = Option.some(h)
        implies modules_linearly_equivalent(h.src, e.src)
} by {
    if linear_equiv_compose(e, linear_equiv_identity(e.src)) = Option.some(h) {
        linear_equiv_compose_identity_right_src(e, h)
        modules_linearly_equivalent_refl(e.src)
        modules_linearly_equivalent(h.src, e.src)
    }
}

/// The destination of a composition of a linear equivalence with the identity on its source
/// agrees with the original's destination via reflexivity of linear equivalence.
theorem linear_equiv_compose_identity_right_modules_linearly_equivalent_refl_dst[R: Ring, M: AddCommGroup, N: AddCommGroup](
    e: LinearEquiv[R, M, N], h: LinearEquiv[R, M, N]
) {
    linear_equiv_compose(e, linear_equiv_identity(e.src)) = Option.some(h)
        implies modules_linearly_equivalent(h.dst, e.dst)
} by {
    if linear_equiv_compose(e, linear_equiv_identity(e.src)) = Option.some(h) {
        linear_equiv_compose_identity_right_dst(e, h)
        modules_linearly_equivalent_refl(e.dst)
        modules_linearly_equivalent(h.dst, e.dst)
    }
}


