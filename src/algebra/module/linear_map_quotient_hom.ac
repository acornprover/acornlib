from algebra.ring.ring import Ring
from algebra.add_comm_group import AddCommGroup
from algebra.module.module import Module
from data.basic.equivalence import QuotientOver
from algebra.module.module_hom import is_linear_map, linear_map_add, linear_map_smul, linear_map_zero
from algebra.module.submodule import linear_map_neg
from data.basic.quotient_algebra import linear_map_kernel_quotient_induced,
    linear_map_kernel_quotient_induced_project, linear_map_kernel_quotient_project,
    linear_map_kernel_quotient_add, linear_map_kernel_quotient_neg,
    linear_map_kernel_quotient_smul, linear_map_kernel_quotient_zero,
    linear_map_kernel_quotient_project_add, linear_map_kernel_quotient_project_neg,
    linear_map_kernel_quotient_project_smul, linear_map_kernel_quotient_project_zero
from data.basic.quotient_algebra_first_iso import linear_map_kernel_quotient_image_eq_iff_project_eq

/// The induced map out of the linear-map-kernel quotient preserves addition.
theorem linear_map_kernel_quotient_induced_add[R: Ring, M: AddCommGroup, N: AddCommGroup](
    src: Module[R, M], dst: Module[R, N], f: M -> N, a: M, b: M
) {
    is_linear_map(src, dst, f) implies
    linear_map_kernel_quotient_induced(src, dst, f,
        linear_map_kernel_quotient_add(src, dst, f,
            linear_map_kernel_quotient_project(src, dst, f, a),
            linear_map_kernel_quotient_project(src, dst, f, b))) =
        linear_map_kernel_quotient_induced(src, dst, f,
            linear_map_kernel_quotient_project(src, dst, f, a)) +
        linear_map_kernel_quotient_induced(src, dst, f,
            linear_map_kernel_quotient_project(src, dst, f, b))
} by {
    if is_linear_map(src, dst, f) {
        linear_map_kernel_quotient_project_add(src, dst, f, a, b)
        linear_map_kernel_quotient_induced_project(src, dst, f, a + b)
        linear_map_kernel_quotient_induced_project(src, dst, f, a)
        linear_map_kernel_quotient_induced_project(src, dst, f, b)
        linear_map_add(src, dst, f, a, b)
        linear_map_kernel_quotient_induced(src, dst, f,
            linear_map_kernel_quotient_add(src, dst, f,
                linear_map_kernel_quotient_project(src, dst, f, a),
                linear_map_kernel_quotient_project(src, dst, f, b))) = f(a) + f(b)
    }
}

/// The induced map out of the linear-map-kernel quotient preserves negation.
theorem linear_map_kernel_quotient_induced_neg[R: Ring, M: AddCommGroup, N: AddCommGroup](
    src: Module[R, M], dst: Module[R, N], f: M -> N, a: M
) {
    is_linear_map(src, dst, f) implies
    linear_map_kernel_quotient_induced(src, dst, f,
        linear_map_kernel_quotient_neg(src, dst, f,
            linear_map_kernel_quotient_project(src, dst, f, a))) =
        -linear_map_kernel_quotient_induced(src, dst, f,
            linear_map_kernel_quotient_project(src, dst, f, a))
} by {
    if is_linear_map(src, dst, f) {
        linear_map_kernel_quotient_project_neg(src, dst, f, a)
        linear_map_kernel_quotient_induced_project(src, dst, f, -a)
        linear_map_kernel_quotient_induced_project(src, dst, f, a)
        linear_map_neg(src, dst, f, a)
        linear_map_kernel_quotient_induced(src, dst, f,
            linear_map_kernel_quotient_neg(src, dst, f,
                linear_map_kernel_quotient_project(src, dst, f, a))) = -f(a)
    }
}

/// The induced map out of the linear-map-kernel quotient preserves scalar multiplication.
theorem linear_map_kernel_quotient_induced_smul[R: Ring, M: AddCommGroup, N: AddCommGroup](
    src: Module[R, M], dst: Module[R, N], f: M -> N, r: R, a: M
) {
    is_linear_map(src, dst, f) implies
    linear_map_kernel_quotient_induced(src, dst, f,
        linear_map_kernel_quotient_smul(src, dst, f, r,
            linear_map_kernel_quotient_project(src, dst, f, a))) =
        dst.smul(r, linear_map_kernel_quotient_induced(src, dst, f,
            linear_map_kernel_quotient_project(src, dst, f, a)))
} by {
    if is_linear_map(src, dst, f) {
        linear_map_kernel_quotient_project_smul(src, dst, f, r, a)
        linear_map_kernel_quotient_induced_project(src, dst, f, src.smul(r, a))
        linear_map_kernel_quotient_induced_project(src, dst, f, a)
        linear_map_smul(src, dst, f, r, a)
        linear_map_kernel_quotient_induced(src, dst, f,
            linear_map_kernel_quotient_smul(src, dst, f, r,
                linear_map_kernel_quotient_project(src, dst, f, a))) = dst.smul(r, f(a))
    }
}

/// The induced map out of the linear-map-kernel quotient preserves zero.
theorem linear_map_kernel_quotient_induced_zero[R: Ring, M: AddCommGroup, N: AddCommGroup](
    src: Module[R, M], dst: Module[R, N], f: M -> N
) {
    is_linear_map(src, dst, f) implies
    linear_map_kernel_quotient_induced(src, dst, f,
        linear_map_kernel_quotient_zero(src, dst, f)) = N.0
} by {
    if is_linear_map(src, dst, f) {
        linear_map_kernel_quotient_project_zero(src, dst, f)
        linear_map_kernel_quotient_induced_project(src, dst, f, M.0)
        linear_map_zero(src, dst, f)
        linear_map_kernel_quotient_induced(src, dst, f,
            linear_map_kernel_quotient_zero(src, dst, f)) = f(M.0)
    }
}

/// A representative lies in the linear-map-kernel quotient fiber over the
/// quotient zero exactly when its image under `f` is the codomain zero.
theorem linear_map_kernel_quotient_project_eq_zero_iff_image_eq_zero[R: Ring, M: AddCommGroup, N: AddCommGroup](
    src: Module[R, M], dst: Module[R, N], f: M -> N, a: M
) {
    is_linear_map(src, dst, f) implies
    (linear_map_kernel_quotient_project(src, dst, f, a) =
        linear_map_kernel_quotient_zero(src, dst, f)) =
        (f(a) = N.0)
} by {
    if is_linear_map(src, dst, f) {
        linear_map_zero(src, dst, f)
        linear_map_kernel_quotient_project_zero(src, dst, f)
        linear_map_kernel_quotient_image_eq_iff_project_eq(src, dst, f, a, M.0)
        (f(a) = N.0) = (linear_map_kernel_quotient_project(src, dst, f, a) =
            linear_map_kernel_quotient_zero(src, dst, f))
    }
}

/// Every value of `f` is attained by the induced map out of the linear-map-kernel quotient.
theorem linear_map_kernel_quotient_induced_surjective_onto_image[R: Ring, M: AddCommGroup, N: AddCommGroup](
    src: Module[R, M], dst: Module[R, N], f: M -> N, a: M
) {
    exists(q: QuotientOver[M]) {
        linear_map_kernel_quotient_induced(src, dst, f, q) = f(a)
    }
} by {
    linear_map_kernel_quotient_induced_project(src, dst, f, a)
}
