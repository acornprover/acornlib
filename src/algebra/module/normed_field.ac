from real import Real, mul_nonneg
from algebra.field.field import Field, inverse_not_zero, mul_not_zero
from algebra.module.normed_add_comm_group import NormedAddCommGroup, norm_non_negative
numerals Real

/// A normed field: a field equipped with a real-valued norm that is multiplicative,
/// `‖a · b‖ = ‖a‖ · ‖b‖`, and which agrees with the additive-group norm structure.
typeclass K: NormedField extends Field, NormedAddCommGroup {
    /// Rule: the norm is multiplicative on the field.
    norm_mul(a: K, b: K) {
        (a * b).norm = a.norm * b.norm
    }
}

/// The norm of the multiplicative identity equals one.
theorem norm_one[K: NormedField] {
    K.1.norm = 1
} by {
    K.1 != K.0
    K.1.norm * K.1.norm.inverse = 1
    K.1.norm * K.1.norm * K.1.norm.inverse = K.1.norm
}

/// The norm of a nonzero element is strictly positive.
theorem norm_pos_of_ne_zero[K: NormedField](a: K) {
    a != K.0 implies a.norm.is_positive
} by {
    if a != K.0 {
        norm_non_negative(a)
        0 < a.norm
        a.norm.is_positive
    }
}

/// The norm of the inverse is the inverse of the norm.
theorem norm_inverse[K: NormedField](a: K) {
    a.norm.inverse = a.inverse.norm
} by {
    if a = K.0 {
        a.norm.inverse = 0
        a.inverse.norm = 0
    } else {
        a * a.inverse = K.1
        (a * a.inverse).norm = K.1.norm
        a.norm * a.inverse.norm = K.1.norm
        K.1.norm = 1
        a.norm * a.inverse.norm = 1
        a.norm != 0
        a.norm * a.norm.inverse = 1
        a.norm.inverse = a.inverse.norm
    }
}

/// The norm of an inverse is `1` divided by the norm.
theorem norm_inverse_div[K: NormedField](a: K) {
    a != K.0 implies a.inverse.norm = 1 / a.norm
} by {
    if a != K.0 {
        norm_inverse(a)
        a.norm.inverse = 1 / a.norm
    }
}

/// The norm of a product with an inverse equals the quotient of the norms.
theorem norm_mul_inverse[K: NormedField](a: K, b: K) {
    (a * b.inverse).norm = a.norm * b.inverse.norm
}

/// The norm of an inverse is non-negative.
theorem norm_inverse_non_negative[K: NormedField](a: K) {
    0 <= a.inverse.norm
} by {
    norm_non_negative(a.inverse)
}

/// The inverse of a nonzero element has strictly positive norm.
theorem norm_inverse_pos[K: NormedField](a: K) {
    a != K.0 implies a.inverse.norm.is_positive
} by {
    if a != K.0 {
        inverse_not_zero(a)
        a.inverse != K.0
        norm_pos_of_ne_zero(a.inverse)
    }
}

/// The product of two norm values is non-negative.
theorem norm_mul_non_negative[K: NormedField](a: K, b: K) {
    0 <= a.norm * b.norm
} by {
    norm_non_negative(a)
    norm_non_negative(b)
    mul_nonneg(a.norm, b.norm)
}

/// A product of nonzero field elements has strictly positive norm.
theorem norm_mul_pos_of_ne_zero[K: NormedField](a: K, b: K) {
    a != K.0 and b != K.0 implies (a * b).norm.is_positive
} by {
    if a != K.0 and b != K.0 {
        mul_not_zero(a, b)
        a * b != K.0
        norm_pos_of_ne_zero(a * b)
    }
}

/// For nonzero denominator, the norm of multiplying by an inverse is the
/// quotient of norms.
theorem norm_mul_inverse_div[K: NormedField](a: K, b: K) {
    b != K.0 implies (a * b.inverse).norm = a.norm / b.norm
} by {
    if b != K.0 {
        norm_mul_inverse(a, b)
        norm_inverse_div(b)
        (a * b.inverse).norm = a.norm / b.norm
    }
}
