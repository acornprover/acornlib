from algebra.ring.ring import Ring
from algebra.add_comm_group import AddCommGroup
from algebra.module.module import Module
from algebra.module.module_hom import ModuleHom, module_hom_trivial, module_hom_trivial_hom_at
from algebra.module.submodule import Submodule, zero_submodule, zero_submodule_contains_eq,
    module_hom_image, module_hom_image_contains_value, module_hom_image_elim

/// A point in the image of the trivial module homomorphism is zero.
theorem module_hom_trivial_image_contains_implies_zero[R: Ring, M: AddCommGroup, N: AddCommGroup](
    src: Module[R, M], dst: Module[R, N], y: N
) {
    module_hom_image(module_hom_trivial(src, dst)).contains(y) implies y = N.0
} by {
    if module_hom_image(module_hom_trivial(src, dst)).contains(y) {
        module_hom_image_elim(module_hom_trivial(src, dst), y)
        let x: M satisfy { module_hom_trivial(src, dst).hom(x) = y }
        module_hom_trivial_hom_at(src, dst, x)
        y = N.0
    }
}

/// Zero lies in the image of the trivial module homomorphism.
theorem module_hom_trivial_image_contains_zero[R: Ring, M: AddCommGroup, N: AddCommGroup](
    src: Module[R, M], dst: Module[R, N]
) {
    module_hom_image(module_hom_trivial(src, dst)).contains(N.0)
} by {
    module_hom_image_contains_value(module_hom_trivial(src, dst), M.0)
    module_hom_trivial_hom_at(src, dst, M.0)
}

/// Membership in the image of the trivial module homomorphism is equality to zero.
theorem module_hom_trivial_image_contains_eq[R: Ring, M: AddCommGroup, N: AddCommGroup](
    src: Module[R, M], dst: Module[R, N], y: N
) {
    module_hom_image(module_hom_trivial(src, dst)).contains(y) = (y = N.0)
} by {
    module_hom_trivial_image_contains_implies_zero(src, dst, y)
    if y = N.0 {
        module_hom_trivial_image_contains_zero(src, dst)
        module_hom_image(module_hom_trivial(src, dst)).contains(y)
    }
}

/// The image of the trivial module homomorphism and the zero submodule share membership at every point.
theorem module_hom_trivial_image_contains_eq_zero_submodule[R: Ring, M: AddCommGroup, N: AddCommGroup](
    src: Module[R, M], dst: Module[R, N], y: N
) {
    module_hom_image(module_hom_trivial(src, dst)).contains(y) = zero_submodule(dst).contains(y)
} by {
    module_hom_trivial_image_contains_eq(src, dst, y)
    zero_submodule_contains_eq(dst, y)
}
