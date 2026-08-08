from algebra.ring.ring import Ring
from algebra.add_comm_group import AddCommGroup
from algebra.module.module import Module
from algebra.module.module_hom import module_hom_trivial, module_hom_trivial_src, module_hom_trivial_dst
from algebra.module.submodule import submodule_eq_of_carrier_contains_at_eq,
    zero_submodule, full_submodule, zero_submodule_carrier_eq, full_submodule_carrier_eq,
    module_hom_kernel, module_hom_image, module_hom_kernel_carrier, module_hom_image_carrier,
    module_hom_trivial_kernel_contains_eq_full_submodule
from algebra.module.module_hom_trivial_image import module_hom_trivial_image_contains_eq_zero_submodule

/// The kernel of the trivial module homomorphism is the full source submodule.
theorem module_hom_trivial_kernel_eq_full_submodule[R: Ring, M: AddCommGroup, N: AddCommGroup](
    src: Module[R, M], dst: Module[R, N]
) {
    module_hom_kernel(module_hom_trivial(src, dst)) = full_submodule(src)
} by {
    let ker = module_hom_kernel(module_hom_trivial(src, dst))
    let full = full_submodule(src)
    module_hom_kernel_carrier(module_hom_trivial(src, dst))
    module_hom_trivial_src(src, dst)
    full_submodule_carrier_eq(src)
    ker.carrier = full.carrier
    forall(x: M) {
        module_hom_trivial_kernel_contains_eq_full_submodule(src, dst, x)
        ker.contains(x) = full.contains(x)
    }
    submodule_eq_of_carrier_contains_at_eq(ker, full)
    ker = full
}

/// The image of the trivial module homomorphism is the zero target submodule.
theorem module_hom_trivial_image_eq_zero_submodule[R: Ring, M: AddCommGroup, N: AddCommGroup](
    src: Module[R, M], dst: Module[R, N]
) {
    module_hom_image(module_hom_trivial(src, dst)) = zero_submodule(dst)
} by {
    let img = module_hom_image(module_hom_trivial(src, dst))
    let z = zero_submodule(dst)
    module_hom_image_carrier(module_hom_trivial(src, dst))
    module_hom_trivial_dst(src, dst)
    zero_submodule_carrier_eq(dst)
    img.carrier = z.carrier
    forall(y: N) {
        module_hom_trivial_image_contains_eq_zero_submodule(src, dst, y)
        img.contains(y) = z.contains(y)
    }
    submodule_eq_of_carrier_contains_at_eq(img, z)
    img = z
}
