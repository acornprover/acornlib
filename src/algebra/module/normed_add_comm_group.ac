from real import Real
from order import lt_lte_trans, not_lt_self
from algebra.add_group import inverse_add
from algebra.add_comm_group import AddCommGroup
from data.basic.set import Set, subset_contains, subset_contains_eq, set_image, maps_into_set_image,
    set_image_contains_witness
numerals Real

/// A normed additive commutative group is an `AddCommGroup` equipped with a real-valued
/// norm satisfying the standard axioms: separation, symmetry under negation, and
/// the triangle inequality.
typeclass G: NormedAddCommGroup extends AddCommGroup {
    /// The norm of a group element.
    norm: G -> Real

    /// Rule: only the zero element has norm zero.
    norm_eq_zero_imp_zero(x: G) {
        x.norm = 0 implies x = G.0
    }

    /// Rule: the zero element has norm zero.
    norm_zero {
        G.0.norm = 0
    }

    /// Rule: the norm is invariant under negation.
    norm_neg(x: G) {
        (-x).norm = x.norm
    }

    /// Rule: the norm satisfies the triangle inequality.
    norm_triangle(x: G, y: G) {
        (x + y).norm <= x.norm + y.norm
    }
}

/// The norm is non-negative.
theorem norm_non_negative[G: NormedAddCommGroup](x: G) {
    0 <= x.norm
} by {
    (x + -x).norm <= x.norm + (-x).norm
    x + -x = G.0
    G.0.norm <= x.norm + x.norm
    0 <= x.norm + x.norm
}

/// The norm is zero exactly at the zero element.
theorem norm_eq_zero_iff[G: NormedAddCommGroup](x: G) {
    (x.norm = 0) = (x = G.0)
} by {
    if x.norm = 0 {
        x = G.0
    }
    if x = G.0 {
        x.norm = 0
    }
}

/// Reverse triangle inequality: the norm difference is bounded by the norm of the difference.
theorem norm_sub_norm_le[G: NormedAddCommGroup](x: G, y: G) {
    x.norm - y.norm <= (x - y).norm
} by {
    x + (-y + y) = x + G.0
    ((x - y) + y).norm = x.norm
    ((x - y) + y).norm <= (x - y).norm + y.norm
    x.norm + -y.norm <= (x - y).norm + y.norm + -y.norm
    (x - y).norm + y.norm + -y.norm = (x - y).norm + (y.norm + -y.norm)
    (x - y).norm + (y.norm + -y.norm) = (x - y).norm + 0
    (x - y).norm + 0 = (x - y).norm
    x.norm + -y.norm = x.norm - y.norm
}

/// The norm of a difference is symmetric in its arguments.
theorem norm_sub_symm[G: NormedAddCommGroup](x: G, y: G) {
    (x - y).norm = (y - x).norm
} by {
    -(x - y) = y - x
}

/// The induced distance function on a normed additive commutative group.
define norm_distance[G: NormedAddCommGroup](x: G, y: G) -> Real {
    (x - y).norm
}

/// The induced distance from a point to itself is zero.
theorem norm_distance_self[G: NormedAddCommGroup](x: G) {
    norm_distance(x, x) = 0
} by {
    x - x = G.0
}

/// The induced distance is symmetric.
theorem norm_distance_symm[G: NormedAddCommGroup](x: G, y: G) {
    norm_distance(x, y) = norm_distance(y, x)
} by {
    (x - y).norm = (y - x).norm
}

/// The induced distance satisfies the triangle inequality.
theorem norm_distance_triangle[G: NormedAddCommGroup](x: G, y: G, z: G) {
    norm_distance(x, z) <= norm_distance(x, y) + norm_distance(y, z)
} by {
    (-y + y) + -z = G.0 + -z
    x + (-y + (y + -z)) = x + -z
    ((x - y) + (y - z)).norm = (x - z).norm
    ((x - y) + (y - z)).norm <= (x - y).norm + (y - z).norm
}

/// The induced distance is non-negative.
theorem norm_distance_non_negative[G: NormedAddCommGroup](x: G, y: G) {
    0 <= norm_distance(x, y)
} by {
    norm_non_negative(x - y)
}

/// The norm of a subtraction equals the norm of its negation argument-swap.
theorem norm_sub_eq_zero_iff[G: NormedAddCommGroup](x: G, y: G) {
    ((x - y).norm = 0) = (x = y)
} by {
    if (x - y).norm = 0 {
        (x + -y) + y = G.0 + y
        x + (-y + y) = x + G.0
        x = y
    }
    if x = y {
        (x - y).norm = G.0.norm
    }
}

/// The induced distance equals zero exactly at coincident points.
theorem norm_distance_eq_zero_iff[G: NormedAddCommGroup](x: G, y: G) {
    (norm_distance(x, y) = 0) = (x = y)
} by {
    norm_sub_eq_zero_iff(x, y)
}

/// Bounding the norm of a sum by bounds on the summands.
theorem norm_add_le_of_le[G: NormedAddCommGroup](x: G, y: G, a: Real, b: Real) {
    x.norm <= a and y.norm <= b implies (x + y).norm <= a + b
} by {
    if x.norm <= a and y.norm <= b {
        x.norm + y.norm <= a + b
    }
}

/// Reverse triangle inequality with arguments swapped: bound via the symmetric difference.
theorem norm_sub_norm_le_swap[G: NormedAddCommGroup](x: G, y: G) {
    y.norm - x.norm <= (x - y).norm
} by {
    norm_sub_norm_le(y, x)
    norm_sub_symm(x, y)
}

/// The norm of a difference equals the norm of the swapped difference.
theorem norm_distance_left_eq[G: NormedAddCommGroup](x: G, y: G) {
    norm_distance(x, y) = (x - y).norm
}

/// The norm of a difference is non-negative.
theorem norm_sub_non_negative[G: NormedAddCommGroup](x: G, y: G) {
    0 <= (x - y).norm
} by {
    norm_non_negative(x - y)
}

/// Distance to the zero element coincides with the norm.
theorem norm_distance_zero[G: NormedAddCommGroup](x: G) {
    norm_distance(x, G.0) = x.norm
} by {
    x + -G.0 = x + G.0
    x - G.0 = x
}

/// Distance from the zero element coincides with the norm.
theorem norm_distance_from_zero[G: NormedAddCommGroup](x: G) {
    norm_distance(G.0, x) = x.norm
} by {
    norm_distance_symm(G.0, x)
    norm_distance_zero(x)
}

/// Bounding the norm of a difference by individual norm bounds via the triangle inequality.
theorem norm_sub_le_add[G: NormedAddCommGroup](x: G, y: G) {
    (x - y).norm <= x.norm + y.norm
} by {
}

/// Distance zero implies equality of points under the induced distance.
theorem norm_distance_zero_imp_eq[G: NormedAddCommGroup](x: G, y: G) {
    norm_distance(x, y) = 0 implies x = y
} by {
    if norm_distance(x, y) = 0 {
        x - y = G.0
        x = y
    }
}

/// True if `y` lies strictly within distance `r` of `x` under the induced norm distance.
define in_norm_open_ball[G: NormedAddCommGroup](x: G, r: Real, y: G) -> Bool {
    norm_distance(y, x) < r
}

/// The open ball of radius `r` centered at `x` under the induced norm distance.
define norm_open_ball[G: NormedAddCommGroup](x: G, r: Real) -> Set[G] {
    Set[G].new(in_norm_open_ball(x, r))
}

/// True if `y` lies within distance `r` of `x` under the induced norm distance.
define in_norm_closed_ball[G: NormedAddCommGroup](x: G, r: Real, y: G) -> Bool {
    norm_distance(y, x) <= r
}

/// The closed ball of radius `r` centered at `x` under the induced norm distance.
define norm_closed_ball[G: NormedAddCommGroup](x: G, r: Real) -> Set[G] {
    Set[G].new(in_norm_closed_ball(x, r))
}

/// Membership in the open norm ball is the strict distance inequality.
theorem norm_open_ball_contains_iff[G: NormedAddCommGroup](x: G, r: Real, y: G) {
    norm_open_ball(x, r).contains(y) = (norm_distance(y, x) < r)
} by {
}

/// Membership in the closed norm ball is the non-strict distance inequality.
theorem norm_closed_ball_contains_iff[G: NormedAddCommGroup](x: G, r: Real, y: G) {
    norm_closed_ball(x, r).contains(y) = (norm_distance(y, x) <= r)
} by {
}

/// Distance to the center governs open-ball membership.
theorem in_norm_open_ball_iff_distance[G: NormedAddCommGroup](x: G, r: Real, y: G) {
    in_norm_open_ball(x, r, y) = (norm_distance(y, x) < r)
}

/// Distance to the center governs closed-ball membership.
theorem in_norm_closed_ball_iff_distance[G: NormedAddCommGroup](x: G, r: Real, y: G) {
    in_norm_closed_ball(x, r, y) = (norm_distance(y, x) <= r)
}

/// The center lies in every open norm ball with positive radius.
theorem center_in_norm_open_ball[G: NormedAddCommGroup](x: G, r: Real) {
    r.is_positive implies norm_open_ball(x, r).contains(x)
} by {
    if r.is_positive {
        0 < r
        norm_distance(x, x) < r
        norm_open_ball(x, r).contains(x)
    }
}

/// The center lies in every closed norm ball with nonnegative radius.
theorem center_in_norm_closed_ball[G: NormedAddCommGroup](x: G, r: Real) {
    not r.is_negative implies norm_closed_ball(x, r).contains(x)
} by {
    if not r.is_negative {
        norm_distance(x, x) <= r
        norm_closed_ball(x, r).contains(x)
    }
}

/// Every open norm ball is contained in the closed norm ball with the same center and radius.
theorem norm_open_ball_subset_closed_ball[G: NormedAddCommGroup](x: G, r: Real) {
    norm_open_ball(x, r).subset(norm_closed_ball(x, r))
} by {
    let b = norm_open_ball(x, r)
    let c = norm_closed_ball(x, r)
    forall(y: G) {
        if b.contains(y) {
            norm_distance(y, x) < r
            norm_distance(y, x) <= r
            c.contains(y)
        }
    }
    subset_contains_eq(b, c)
    b.subset(c)
}

/// No point lies in an open norm ball whose radius is not positive.
theorem norm_open_ball_nonpositive_radius_empty[G: NormedAddCommGroup](x: G, r: Real, y: G) {
    not r.is_positive implies not norm_open_ball(x, r).contains(y)
} by {
    if not r.is_positive {
        let d = norm_distance(y, x)
        0 <= d
        if norm_open_ball(x, r).contains(y) {
            d < r
            if r <= 0 {
                lt_lte_trans(d, r, 0)
                d < 0
                false
            } else {
                r.is_positive
                false
            }
        }
    }
}

/// No point lies in a closed norm ball whose radius is negative.
theorem norm_closed_ball_negative_radius_empty[G: NormedAddCommGroup](x: G, r: Real, y: G) {
    r.is_negative implies not norm_closed_ball(x, r).contains(y)
} by {
    if r.is_negative {
        let d = norm_distance(y, x)
        0 <= d
        if norm_closed_ball(x, r).contains(y) {
            d <= r
            lt_lte_trans(r, 0, d)
            lt_lte_trans(r, d, r)
            r < r
            not_lt_self(r)
            false
        }
    }
}

/// Open norm-ball membership is symmetric in the center and the point.
theorem norm_open_ball_contains_symmetric[G: NormedAddCommGroup](x: G, r: Real, y: G) {
    norm_open_ball(x, r).contains(y) = norm_open_ball(y, r).contains(x)
} by {
    norm_distance(y, x) = norm_distance(x, y)
}

/// Closed norm-ball membership is symmetric in the center and the point.
theorem norm_closed_ball_contains_symmetric[G: NormedAddCommGroup](x: G, r: Real, y: G) {
    norm_closed_ball(x, r).contains(y) = norm_closed_ball(y, r).contains(x)
} by {
    norm_distance(y, x) = norm_distance(x, y)
}

/// True if there is a closed norm ball that contains the entire set `s`.
define norm_is_bounded[G: NormedAddCommGroup](s: Set[G]) -> Bool {
    exists(x: G, r: Real) {
        s.subset(norm_closed_ball(x, r))
    }
}

/// A closed norm ball is norm-bounded.
theorem norm_closed_ball_is_bounded[G: NormedAddCommGroup](x: G, r: Real) {
    norm_is_bounded(norm_closed_ball(x, r))
} by {
    exists(c: G, rr: Real) { norm_closed_ball(x, r).subset(norm_closed_ball(c, rr)) }
}

/// Every subset of a norm-bounded set is norm-bounded.
theorem subset_of_norm_bounded_is_norm_bounded[G: NormedAddCommGroup](s: Set[G], t: Set[G]) {
    s.subset(t) and norm_is_bounded(t) implies norm_is_bounded(s)
} by {
    if s.subset(t) and norm_is_bounded(t) {
        let x: G satisfy {
            exists(r: Real) { t.subset(norm_closed_ball(x, r)) }
        }
        let r: Real satisfy {
            t.subset(norm_closed_ball(x, r))
        }
        let b = norm_closed_ball(x, r)
        forall(y: G) {
            if s.contains(y) {
                subset_contains(s, t, y)
                t.contains(y)
                subset_contains(t, b, y)
                b.contains(y)
            }
        }
        subset_contains_eq(s, b)
        s.subset(b)
        norm_is_bounded(s)
    }
}

/// A singleton set is norm-bounded.
theorem norm_singleton_is_bounded[G: NormedAddCommGroup](x: G) {
    norm_is_bounded(Set[G].singleton(x))
} by {
    let s = Set[G].singleton(x)
    let b = norm_closed_ball(x, 0)
    forall(y: G) {
        if s.contains(y) {
            s.contains(y) = (x = y)
            norm_distance(y, x) = 0
            norm_distance(y, x) <= 0
            b.contains(y)
        }
    }
    subset_contains_eq(s, b)
    s.subset(b)
    norm_is_bounded(s)
}

/// An open norm ball is norm-bounded.
theorem norm_open_ball_is_bounded[G: NormedAddCommGroup](x: G, r: Real) {
    norm_is_bounded(norm_open_ball(x, r))
} by {
    let u = norm_open_ball(x, r)
    let b = norm_closed_ball(x, r)
    norm_open_ball_subset_closed_ball(x, r)
    norm_closed_ball_is_bounded(x, r)
    subset_of_norm_bounded_is_norm_bounded(u, b)
}

/// Norm distance is invariant under translating both arguments by the same element.
theorem norm_distance_add_right[G: NormedAddCommGroup](x: G, y: G, z: G) {
    norm_distance(x + z, y + z) = norm_distance(x, y)
} by {
    inverse_add(y, z)
    (z + -z) + -y = G.0 + -y
    x + (z + (-z + -y)) = x + -y
}

/// Translating the center and point by the same element preserves open norm-ball membership.
theorem norm_open_ball_contains_add_right_iff[G: NormedAddCommGroup](x: G, r: Real, y: G, z: G) {
    norm_open_ball(x + z, r).contains(y + z) = norm_open_ball(x, r).contains(y)
} by {
    norm_distance_add_right(y, x, z)
}

/// Translating the center and point by the same element preserves closed norm-ball membership.
theorem norm_closed_ball_contains_add_right_iff[G: NormedAddCommGroup](x: G, r: Real, y: G, z: G) {
    norm_closed_ball(x + z, r).contains(y + z) = norm_closed_ball(x, r).contains(y)
} by {
    norm_distance_add_right(y, x, z)
}

/// Translation by a fixed element.
define norm_translate[G: NormedAddCommGroup](z: G, x: G) -> G {
    x + z
}

/// Translating a set by a fixed element.
define norm_translate_set[G: NormedAddCommGroup](s: Set[G], z: G) -> Set[G] {
    set_image(s, norm_translate(z))
}

/// Translating an element of a set gives an element of the translated set.
theorem norm_translate_set_contains_translate[G: NormedAddCommGroup](s: Set[G], z: G, x: G) {
    s.contains(x) implies norm_translate_set(s, z).contains(x + z)
} by {
    if s.contains(x) {
        maps_into_set_image(s, norm_translate(z), x)
        norm_translate_set(s, z).contains(x + z)
    }
}

/// Translating an open ball translates its center.
theorem norm_translate_open_ball_subset[G: NormedAddCommGroup](x: G, r: Real, z: G) {
    norm_translate_set(norm_open_ball(x, r), z).subset(norm_open_ball(x + z, r))
} by {
    let s = norm_translate_set(norm_open_ball(x, r), z)
    let b = norm_open_ball(x + z, r)
    forall(y: G) {
        if s.contains(y) {
            set_image_contains_witness(norm_open_ball(x, r), norm_translate(z), y)
            let w: G satisfy {
                norm_open_ball(x, r).contains(w) and y = norm_translate(z, w)
            }
            b.contains(w + z)
            b.contains(y)
        }
    }
    subset_contains_eq(s, b)
    s.subset(b)
}

/// Translating a closed ball translates its center.
theorem norm_translate_closed_ball_subset[G: NormedAddCommGroup](x: G, r: Real, z: G) {
    norm_translate_set(norm_closed_ball(x, r), z).subset(norm_closed_ball(x + z, r))
} by {
    let s = norm_translate_set(norm_closed_ball(x, r), z)
    let b = norm_closed_ball(x + z, r)
    forall(y: G) {
        if s.contains(y) {
            set_image_contains_witness(norm_closed_ball(x, r), norm_translate(z), y)
            let w: G satisfy {
                norm_closed_ball(x, r).contains(w) and y = norm_translate(z, w)
            }
            b.contains(w + z)
            b.contains(y)
        }
    }
    subset_contains_eq(s, b)
    s.subset(b)
}
