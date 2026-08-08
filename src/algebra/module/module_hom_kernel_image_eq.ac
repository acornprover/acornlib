from algebra.ring.ring import Ring
from algebra.add_comm_group import AddCommGroup
from algebra.module.module_hom import ModuleHom
from data.basic.functions import is_injective_fn, is_surjective_fn
from algebra.module.submodule import module_hom_kernel, module_hom_image,
    zero_submodule, full_submodule,
    module_hom_kernel_carrier, module_hom_image_carrier,
    zero_submodule_carrier_eq, full_submodule_carrier_eq,
    zero_submodule_subset, submodule_subset_full, submodule_subset,
    submodule_subset_antisymm, module_hom_kernel_subset_zero_of_injective,
    full_submodule_subset_module_hom_image_of_surjective,
    module_hom_image_of_image_eq_full

/// The zero submodule of the source is contained in the kernel submodule of a
/// bundled module homomorphism.
theorem zero_submodule_subset_module_hom_kernel[R: Ring, M: AddCommGroup, N: AddCommGroup](
    f: ModuleHom[R, M, N]
) {
    submodule_subset(zero_submodule(f.src), module_hom_kernel(f))
} by {
    module_hom_kernel_carrier(f)
    zero_submodule_subset(module_hom_kernel(f))
}

/// The image submodule of a bundled module homomorphism is contained in the
/// full submodule of the destination.
theorem module_hom_image_subset_full[R: Ring, M: AddCommGroup, N: AddCommGroup](
    f: ModuleHom[R, M, N]
) {
    submodule_subset(module_hom_image(f), full_submodule(f.dst))
} by {
    module_hom_image_carrier(f)
    submodule_subset_full(module_hom_image(f))
}

/// An injective bundled module homomorphism has the zero submodule as its kernel.
theorem module_hom_kernel_eq_zero_of_injective[R: Ring, M: AddCommGroup, N: AddCommGroup](
    f: ModuleHom[R, M, N]
) {
    is_injective_fn(f.hom) implies module_hom_kernel(f) = zero_submodule(f.src)
} by {
    if is_injective_fn(f.hom) {
        module_hom_kernel_subset_zero_of_injective(f)
        zero_submodule_subset_module_hom_kernel(f)
        module_hom_kernel_carrier(f)
        zero_submodule_carrier_eq(f.src)
        submodule_subset_antisymm(module_hom_kernel(f), zero_submodule(f.src))
        module_hom_kernel(f) = zero_submodule(f.src)
    }
}

/// A surjective bundled module homomorphism has the full submodule as its image.
theorem module_hom_image_eq_full_of_surjective[R: Ring, M: AddCommGroup, N: AddCommGroup](
    f: ModuleHom[R, M, N]
) {
    is_surjective_fn(f.hom) implies module_hom_image(f) = full_submodule(f.dst)
} by {
    if is_surjective_fn(f.hom) {
        full_submodule_subset_module_hom_image_of_surjective(f)
        module_hom_image_subset_full(f)
        module_hom_image_carrier(f)
        full_submodule_carrier_eq(f.dst)
        submodule_subset_antisymm(module_hom_image(f), full_submodule(f.dst))
        module_hom_image(f) = full_submodule(f.dst)
    }
}

/// A bundled module homomorphism whose image is full has a preimage for every target point.
theorem module_hom_forall_preimage_of_image_eq_full[R: Ring, M: AddCommGroup, N: AddCommGroup](
    f: ModuleHom[R, M, N]
) {
    module_hom_image(f) = full_submodule(f.dst) implies forall(y: N) {
        exists(x: M) {
            f.hom(x) = y
        }
    }
} by {
    if module_hom_image(f) = full_submodule(f.dst) {
        forall(y: N) {
            module_hom_image_of_image_eq_full(f, y)
            exists(x: M) {
                f.hom(x) = y
            }
        }
    }
}

/// A bundled module homomorphism whose image is full is surjective.
theorem module_hom_surjective_of_image_eq_full[R: Ring, M: AddCommGroup, N: AddCommGroup](
    f: ModuleHom[R, M, N]
) {
    module_hom_image(f) = full_submodule(f.dst) implies is_surjective_fn(f.hom)
} by {
    if module_hom_image(f) = full_submodule(f.dst) {
        module_hom_forall_preimage_of_image_eq_full(f)
        forall(y: N) {
            exists(x: M) {
                f.hom(x) = y
            }
        }
        is_surjective_fn(f.hom)
    }
}
