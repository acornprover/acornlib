from algebra.ring.ring import Ring
from algebra.add_comm_group import AddCommGroup
from data.basic.functions import is_surjective_fn
from algebra.module.module_hom import ModuleHom, module_hom_compose, module_hom_compose_dst,
    module_hom_compose_hom_at
from algebra.module.submodule import module_hom_image, module_hom_image_carrier,
    module_hom_image_elim, module_hom_image_contains_value, submodule_subset,
    submodule_subset_antisymm

/// A point in the image of a bundled composition lies in the image of the outer homomorphism.
theorem module_hom_compose_image_contains_implies_image_outer[R: Ring, M: AddCommGroup, N: AddCommGroup, K: AddCommGroup](
    f: ModuleHom[R, N, K], g: ModuleHom[R, M, N], h: ModuleHom[R, M, K], y: K
) {
    module_hom_compose(f, g) = Option.some(h) and module_hom_image(h).contains(y)
        implies module_hom_image(f).contains(y)
} by {
    if module_hom_compose(f, g) = Option.some(h) and module_hom_image(h).contains(y) {
        module_hom_image_elim(h, y)
        let x: M satisfy { h.hom(x) = y }
        module_hom_compose_hom_at(f, g, h, x)
        module_hom_image_contains_value(f, g.hom(x))
        module_hom_image(f).contains(y)
    }
}

/// The image submodule of a bundled composition is contained in the image submodule of the outer homomorphism.
theorem module_hom_compose_image_subset_image_outer[R: Ring, M: AddCommGroup, N: AddCommGroup, K: AddCommGroup](
    f: ModuleHom[R, N, K], g: ModuleHom[R, M, N], h: ModuleHom[R, M, K]
) {
    module_hom_compose(f, g) = Option.some(h)
        implies submodule_subset(module_hom_image(h), module_hom_image(f))
} by {
    if module_hom_compose(f, g) = Option.some(h) {
        forall(y: K) {
            if module_hom_image(h).contains(y) {
                module_hom_compose_image_contains_implies_image_outer(f, g, h, y)
                module_hom_image(f).contains(y)
            }
        }
    }
}

/// If the inner homomorphism is surjective, every point in the outer image lies in the composition image.
theorem module_hom_image_outer_contains_implies_compose_image_of_surjective_inner[R: Ring, M: AddCommGroup, N: AddCommGroup, K: AddCommGroup](
    f: ModuleHom[R, N, K], g: ModuleHom[R, M, N], h: ModuleHom[R, M, K], y: K
) {
    module_hom_compose(f, g) = Option.some(h) and is_surjective_fn(g.hom)
        and module_hom_image(f).contains(y)
        implies module_hom_image(h).contains(y)
} by {
    if module_hom_compose(f, g) = Option.some(h) and is_surjective_fn(g.hom)
        and module_hom_image(f).contains(y) {
        module_hom_image_elim(f, y)
        let z: N satisfy { f.hom(z) = y }
        is_surjective_fn(g.hom) = forall(w: N) {
            exists(x: M) {
                g.hom(x) = w
            }
        }
        let x: M satisfy { g.hom(x) = z }
        module_hom_compose_hom_at(f, g, h, x)
        h.hom(x) = f.hom(g.hom(x))
        module_hom_image_contains_value(h, x)
        module_hom_image(h).contains(y)
    }
}

/// If the inner homomorphism is surjective, the outer image is contained in the composition image.
theorem module_hom_image_outer_subset_compose_image_of_surjective_inner[R: Ring, M: AddCommGroup, N: AddCommGroup, K: AddCommGroup](
    f: ModuleHom[R, N, K], g: ModuleHom[R, M, N], h: ModuleHom[R, M, K]
) {
    module_hom_compose(f, g) = Option.some(h) and is_surjective_fn(g.hom)
        implies submodule_subset(module_hom_image(f), module_hom_image(h))
} by {
    if module_hom_compose(f, g) = Option.some(h) and is_surjective_fn(g.hom) {
        forall(y: K) {
            if module_hom_image(f).contains(y) {
                module_hom_image_outer_contains_implies_compose_image_of_surjective_inner(f, g, h, y)
                module_hom_image(h).contains(y)
            }
        }
    }
}

/// If the inner homomorphism is surjective, the image of a bundled composition is exactly the outer image.
theorem module_hom_compose_image_eq_image_outer_of_surjective_inner[R: Ring, M: AddCommGroup, N: AddCommGroup, K: AddCommGroup](
    f: ModuleHom[R, N, K], g: ModuleHom[R, M, N], h: ModuleHom[R, M, K]
) {
    module_hom_compose(f, g) = Option.some(h) and is_surjective_fn(g.hom)
        implies module_hom_image(h) = module_hom_image(f)
} by {
    if module_hom_compose(f, g) = Option.some(h) and is_surjective_fn(g.hom) {
        let image_h = module_hom_image(h)
        let image_f = module_hom_image(f)
        module_hom_compose_image_subset_image_outer(f, g, h)
        module_hom_image_outer_subset_compose_image_of_surjective_inner(f, g, h)
        submodule_subset(image_f, image_h)
        module_hom_image_carrier(h)
        module_hom_compose_dst(f, g, h)
        module_hom_image_carrier(f)
        image_h.carrier = image_f.carrier
        submodule_subset_antisymm(image_h, image_f)
        image_h = image_f
        module_hom_image(h) = module_hom_image(f)
    }
}
