from algebra.ring.ring import Ring
from algebra.add_comm_group import AddCommGroup
from algebra.module.module_hom import ModuleHom
from data.basic.functions import is_injective_fn, is_surjective_fn
from algebra.module.submodule import submodule_subset, submodule_subset_antisymm,
    zero_submodule, full_submodule,
    module_hom_kernel, module_hom_image,
    module_hom_kernel_carrier, module_hom_image_carrier,
    zero_submodule_carrier_eq, full_submodule_carrier_eq,
    zero_submodule_subset, module_hom_kernel_subset_zero_of_injective,
    full_submodule_subset_module_hom_image_of_surjective,
    module_hom_image_subset_full, module_hom_injective_of_kernel_eq_zero,
    module_hom_image_of_image_eq_full

/// An injective bundled module homomorphism has zero kernel submodule.
theorem module_hom_kernel_eq_zero_of_injective[R: Ring, M: AddCommGroup, N: AddCommGroup](
    f: ModuleHom[R, M, N]
) {
    is_injective_fn(f.hom) implies module_hom_kernel(f) = zero_submodule(f.src)
} by {
    if is_injective_fn(f.hom) {
        module_hom_kernel_carrier(f)
        zero_submodule_carrier_eq(f.src)
        module_hom_kernel_subset_zero_of_injective(f)
        zero_submodule_subset(module_hom_kernel(f))
        submodule_subset(zero_submodule(f.src), module_hom_kernel(f))
        submodule_subset_antisymm(module_hom_kernel(f), zero_submodule(f.src))
        module_hom_kernel(f) = zero_submodule(f.src)
    }
}

/// A surjective bundled module homomorphism has full image submodule.
theorem module_hom_image_eq_full_of_surjective[R: Ring, M: AddCommGroup, N: AddCommGroup](
    f: ModuleHom[R, M, N]
) {
    is_surjective_fn(f.hom) implies module_hom_image(f) = full_submodule(f.dst)
} by {
    if is_surjective_fn(f.hom) {
        module_hom_image_carrier(f)
        full_submodule_carrier_eq(f.dst)
        full_submodule_subset_module_hom_image_of_surjective(f)
        module_hom_image_subset_full(f)
        submodule_subset_antisymm(module_hom_image(f), full_submodule(f.dst))
        module_hom_image(f) = full_submodule(f.dst)
    }
}

/// A bundled module homomorphism is injective exactly when its kernel submodule is zero.
theorem module_hom_injective_iff_kernel_eq_zero[R: Ring, M: AddCommGroup, N: AddCommGroup](
    f: ModuleHom[R, M, N]
) {
    is_injective_fn(f.hom) = (module_hom_kernel(f) = zero_submodule(f.src))
} by {
    if is_injective_fn(f.hom) {
        module_hom_kernel_eq_zero_of_injective(f)
        module_hom_kernel(f) = zero_submodule(f.src)
    }
    if module_hom_kernel(f) = zero_submodule(f.src) {
        module_hom_injective_of_kernel_eq_zero(f)
        is_injective_fn(f.hom)
    }
}

/// A bundled module homomorphism is surjective exactly when its image submodule is full.
theorem module_hom_surjective_iff_image_eq_full[R: Ring, M: AddCommGroup, N: AddCommGroup](
    f: ModuleHom[R, M, N]
) {
    is_surjective_fn(f.hom) = (module_hom_image(f) = full_submodule(f.dst))
} by {
    if is_surjective_fn(f.hom) {
        module_hom_image_eq_full_of_surjective(f)
        module_hom_image(f) = full_submodule(f.dst)
    }
    if module_hom_image(f) = full_submodule(f.dst) {
        forall(y: N) {
            module_hom_image_of_image_eq_full(f, y)
            exists(x: M) { f.hom(x) = y }
        }
        is_surjective_fn(f.hom)
    }
}
