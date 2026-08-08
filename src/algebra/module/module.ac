from algebra.ring.ring import Ring
from algebra.add_comm_group import AddCommGroup
from algebra.add_group import left_cancel, right_cancel

/// True if scalar multiplication distributes over scalar addition.
define module_smul_add_left_constraint[R: Ring, M: AddCommGroup](smul: R -> M -> M) -> Bool {
    forall(r: R, s: R, x: M) {
        smul(r + s, x) = smul(r, x) + smul(s, x)
    }
}

/// True if scalar multiplication distributes over module addition.
define module_smul_add_right_constraint[R: Ring, M: AddCommGroup](smul: R -> M -> M) -> Bool {
    forall(r: R, x: M, y: M) {
        smul(r, x + y) = smul(r, x) + smul(r, y)
    }
}

/// True if scalar multiplication is compatible with ring multiplication.
define module_smul_assoc_constraint[R: Ring, M: AddCommGroup](smul: R -> M -> M) -> Bool {
    forall(r: R, s: R, x: M) {
        smul(r * s, x) = smul(r, smul(s, x))
    }
}

/// True if the ring identity acts as the identity scalar.
define module_smul_one_constraint[R: Ring, M: AddCommGroup](smul: R -> M -> M) -> Bool {
    forall(x: M) {
        smul(R.1, x) = x
    }
}

/// True if a scalar action makes M a left R-module.
define is_module_action[R: Ring, M: AddCommGroup](smul: R -> M -> M) -> Bool {
    module_smul_add_left_constraint(smul)
    and module_smul_add_right_constraint(smul)
    and module_smul_assoc_constraint(smul)
    and module_smul_one_constraint(smul)
}

/// A left R-module: an additive abelian group together with an action of R that is
/// distributive on both sides, compatible with ring multiplication, and unital.
structure Module[R: Ring, M: AddCommGroup] {
    /// Scalar multiplication: combines a ring element and a module element.
    smul: R -> M -> M
} constraint {
    is_module_action(smul)
}

/// Module extensionality: two modules are equal when their scalar actions agree pointwise.
theorem module_ext[R: Ring, M: AddCommGroup](a: Module[R, M], b: Module[R, M]) {
    (forall(r: R, x: M) { a.smul(r, x) = b.smul(r, x) }) implies a = b
} by {
    if forall(r: R, x: M) { a.smul(r, x) = b.smul(r, x) } {
        forall(r: R) {
            forall(x: M) {
                a.smul(r, x) = b.smul(r, x)
            }
            a.smul(r) = b.smul(r)
        }
        a.smul = b.smul
    }
}

/// Scalar multiplication distributes over scalar addition.
theorem module_smul_add_left[R: Ring, M: AddCommGroup](m: Module[R, M], r: R, s: R, x: M) {
    m.smul(r + s, x) = m.smul(r, x) + m.smul(s, x)
} by {
    is_module_action(m.smul) =
        (module_smul_add_left_constraint(m.smul)
         and module_smul_add_right_constraint(m.smul)
         and module_smul_assoc_constraint(m.smul)
         and module_smul_one_constraint(m.smul))
    module_smul_add_left_constraint(m.smul) = forall(a: R, b: R, y: M) {
        m.smul(a + b, y) = m.smul(a, y) + m.smul(b, y)
    }
}

/// Scalar multiplication distributes over module addition.
theorem module_smul_add_right[R: Ring, M: AddCommGroup](m: Module[R, M], r: R, x: M, y: M) {
    m.smul(r, x + y) = m.smul(r, x) + m.smul(r, y)
} by {
    is_module_action(m.smul) =
        (module_smul_add_left_constraint(m.smul)
         and module_smul_add_right_constraint(m.smul)
         and module_smul_assoc_constraint(m.smul)
         and module_smul_one_constraint(m.smul))
    module_smul_add_right_constraint(m.smul) = forall(a: R, u: M, v: M) {
        m.smul(a, u + v) = m.smul(a, u) + m.smul(a, v)
    }
}

/// Scalar multiplication is compatible with ring multiplication.
theorem module_smul_assoc[R: Ring, M: AddCommGroup](m: Module[R, M], r: R, s: R, x: M) {
    m.smul(r * s, x) = m.smul(r, m.smul(s, x))
} by {
    is_module_action(m.smul) =
        (module_smul_add_left_constraint(m.smul)
         and module_smul_add_right_constraint(m.smul)
         and module_smul_assoc_constraint(m.smul)
         and module_smul_one_constraint(m.smul))
    module_smul_assoc_constraint(m.smul) = forall(a: R, b: R, y: M) {
        m.smul(a * b, y) = m.smul(a, m.smul(b, y))
    }
}

/// The ring identity acts as the identity scalar.
theorem module_smul_one[R: Ring, M: AddCommGroup](m: Module[R, M], x: M) {
    m.smul(R.1, x) = x
} by {
    module_smul_one_constraint(m.smul)
}

/// The zero scalar annihilates every module element.
theorem module_smul_zero_left[R: Ring, M: AddCommGroup](m: Module[R, M], x: M) {
    m.smul(R.0, x) = M.0
} by {
    module_smul_add_left(m, R.0, R.0, x)
    left_cancel(m.smul(R.0, x), m.smul(R.0, x), M.0)
}

/// Any scalar annihilates the zero module element.
theorem module_smul_zero_right[R: Ring, M: AddCommGroup](m: Module[R, M], r: R) {
    m.smul(r, M.0) = M.0
} by {
    module_smul_add_right(m, r, M.0, M.0)
    left_cancel(m.smul(r, M.0), m.smul(r, M.0), M.0)
}

/// Negating the scalar negates the action.
theorem module_smul_neg_left[R: Ring, M: AddCommGroup](m: Module[R, M], r: R, x: M) {
    m.smul(-r, x) = -m.smul(r, x)
} by {
    module_smul_add_left(m, -r, r, x)
    module_smul_zero_left(m, x)
    right_cancel(m.smul(r, x), m.smul(-r, x), -m.smul(r, x))
}

/// Negating the module element negates the action.
theorem module_smul_neg_right[R: Ring, M: AddCommGroup](m: Module[R, M], r: R, x: M) {
    m.smul(r, -x) = -m.smul(r, x)
} by {
    module_smul_add_right(m, r, -x, x)
    module_smul_zero_right(m, r)
    right_cancel(m.smul(r, x), m.smul(r, -x), -m.smul(r, x))
}

/// Ring multiplication as a scalar action of a ring on itself.
let ring_mul_smul[R: Ring]: R -> R -> R = function(r: R, s: R) {
    r * s
}

/// A ring's multiplication makes it a module over itself.
theorem ring_mul_is_module_action[R: Ring] {
    is_module_action(ring_mul_smul[R])
} by {
    forall(r: R, s: R, x: R) {
        (r + s) * x = r * x + s * x
        ring_mul_smul[R](r + s, x) = ring_mul_smul[R](r, x) + ring_mul_smul[R](s, x)
    }
    module_smul_add_left_constraint(ring_mul_smul[R])

    forall(r: R, x: R, y: R) {
        r * (x + y) = r * x + r * y
        ring_mul_smul[R](r, x + y) = ring_mul_smul[R](r, x) + ring_mul_smul[R](r, y)
    }
    module_smul_add_right_constraint(ring_mul_smul[R])

    forall(r: R, s: R, x: R) {
        ring_mul_smul[R](r * s, x) = ring_mul_smul[R](r, ring_mul_smul[R](s, x))
    }
    module_smul_assoc_constraint(ring_mul_smul[R])

    forall(x: R) {
        ring_mul_smul[R](R.1, x) = x
    }
    module_smul_one_constraint(ring_mul_smul[R])
}

/// A ring viewed as a module over itself.
let ring_as_module[R: Ring]: Module[R, R] satisfy {
    Module.new(ring_mul_smul[R]) = Option.some(ring_as_module)
}

/// The scalar action on the ring-as-module is ring multiplication.
theorem ring_as_module_smul[R: Ring](r: R, s: R) {
    ring_as_module[R].smul(r, s) = r * s
} by {
    ring_as_module[R].smul = ring_mul_smul[R]
}
