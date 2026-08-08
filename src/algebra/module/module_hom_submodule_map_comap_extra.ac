/// Additional bundled module-hom submodule map/comap consumers.

from algebra.ring.ring import Ring
from algebra.add_comm_group import AddCommGroup
from algebra.module.module import Module
from algebra.module.module_hom import ModuleHom, module_hom_is_linear_map, module_hom_identity,
    module_hom_identity_src, module_hom_identity_hom
from algebra.module.submodule import Submodule, submodule_subset, zero_submodule
from algebra.module.module_hom_submodule_map_comap import module_hom_submodule_map,
    module_hom_submodule_comap
from algebra.module.submodule_map_comap_extra import linear_map_submodule_image_subset_of_maps_into,
    linear_map_submodule_image_subset_iff_maps_into,
    linear_map_submodule_subset_preimage_image,
    linear_map_submodule_image_preimage_subset,
    linear_map_preimage_submodule_identity,
    linear_map_submodule_image_zero_eq_zero

/// A pointwise maps-into condition bounds the bundled submodule image.
theorem module_hom_submodule_map_subset_of_maps_into[R: Ring, M: AddCommGroup, N: AddCommGroup](
    f: ModuleHom[R, M, N], s: Submodule[R, M], t: Submodule[R, N]
) {
    t.carrier = f.dst and (forall(x: M) { s.contains(x) implies t.contains(f.hom(x)) })
        implies submodule_subset(module_hom_submodule_map(f, s), t)
} by {
    if t.carrier = f.dst and forall(x: M) { s.contains(x) implies t.contains(f.hom(x)) } {
        linear_map_submodule_image_subset_of_maps_into(f.src, f.dst, f.hom, s, t)
        submodule_subset(module_hom_submodule_map(f, s), t)
    }
}

/// Bounding the bundled submodule image is equivalent to checking images of source members.
theorem module_hom_submodule_map_subset_iff_maps_into[R: Ring, M: AddCommGroup, N: AddCommGroup](
    f: ModuleHom[R, M, N], s: Submodule[R, M], t: Submodule[R, N]
) {
    t.carrier = f.dst implies
        (submodule_subset(module_hom_submodule_map(f, s), t) =
            forall(x: M) { s.contains(x) implies t.contains(f.hom(x)) })
} by {
    if t.carrier = f.dst {
        linear_map_submodule_image_subset_iff_maps_into(f.src, f.dst, f.hom, s, t)
        submodule_subset(module_hom_submodule_map(f, s), t) =
            forall(x: M) { s.contains(x) implies t.contains(f.hom(x)) }
    }
}

/// The unit inequality of bundled submodule map/comap: `s <= comap (map s)`.
theorem module_hom_submodule_subset_comap_map[R: Ring, M: AddCommGroup, N: AddCommGroup](
    f: ModuleHom[R, M, N], s: Submodule[R, M]
) {
    s.carrier = f.src implies submodule_subset(s, module_hom_submodule_comap(f, module_hom_submodule_map(f, s)))
} by {
    if s.carrier = f.src {
        module_hom_is_linear_map(f)
        linear_map_submodule_subset_preimage_image(f.src, f.dst, f.hom, s)
        submodule_subset(s, module_hom_submodule_comap(f, module_hom_submodule_map(f, s)))
    }
}

/// The counit inequality of bundled submodule map/comap: `map (comap t) <= t`.
theorem module_hom_submodule_map_comap_subset[R: Ring, M: AddCommGroup, N: AddCommGroup](
    f: ModuleHom[R, M, N], t: Submodule[R, N]
) {
    t.carrier = f.dst implies submodule_subset(module_hom_submodule_map(f, module_hom_submodule_comap(f, t)), t)
} by {
    if t.carrier = f.dst {
        module_hom_is_linear_map(f)
        linear_map_submodule_image_preimage_subset(f.src, f.dst, f.hom, t)
        submodule_subset(module_hom_submodule_map(f, module_hom_submodule_comap(f, t)), t)
    }
}

/// Comapping along the identity module homomorphism fixes every submodule.
theorem module_hom_submodule_comap_identity[R: Ring, M: AddCommGroup](
    src: Module[R, M], t: Submodule[R, M]
) {
    t.carrier = src implies module_hom_submodule_comap(module_hom_identity(src), t) = t
} by {
    if t.carrier = src {
        module_hom_identity_src(src)
        module_hom_identity_hom(src)
        linear_map_preimage_submodule_identity(src, t)
        module_hom_submodule_comap(module_hom_identity(src), t) = t
    }
}

/// Mapping the zero source submodule by a bundled module homomorphism gives the zero target submodule.
theorem module_hom_submodule_map_zero_eq_zero[R: Ring, M: AddCommGroup, N: AddCommGroup](
    f: ModuleHom[R, M, N]
) {
    module_hom_submodule_map(f, zero_submodule(f.src)) = zero_submodule(f.dst)
} by {
    module_hom_is_linear_map(f)
    linear_map_submodule_image_zero_eq_zero(f.src, f.dst, f.hom)
    module_hom_submodule_map(f, zero_submodule(f.src)) = zero_submodule(f.dst)
}
