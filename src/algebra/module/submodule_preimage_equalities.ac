from algebra.ring.ring import Ring
from algebra.add_comm_group import AddCommGroup
from algebra.module.module import Module
from data.basic.functions import compose
from algebra.module.module_hom import is_linear_map, compose_is_linear_map
from algebra.module.submodule import submodule_eq_of_carrier_contains_at_eq,
    zero_submodule, full_submodule,
    linear_map_preimage_submodule, linear_map_preimage_submodule_carrier_eq,
    linear_map_preimage_submodule_contains_eq,
    linear_map_kernel_submodule, linear_map_kernel_submodule_carrier_eq,
    linear_map_preimage_submodule_zero_eq_kernel_at,
    linear_map_preimage_submodule_full_contains_at,
    full_submodule_carrier_eq, linear_map_kernel_submodule_map_eq_zero_of_contains,
    linear_map_kernel_submodule_contains_of_map_eq_zero

/// The preimage of the zero submodule under a linear map is its kernel submodule.
theorem linear_map_preimage_submodule_zero_eq_kernel[R: Ring, M: AddCommGroup, N: AddCommGroup](
    src: Module[R, M], dst: Module[R, N], f: M -> N
) {
    is_linear_map(src, dst, f) implies
        linear_map_preimage_submodule(src, zero_submodule(dst), f) =
            linear_map_kernel_submodule(src, dst, f)
} by {
    if is_linear_map(src, dst, f) {
        let pre = linear_map_preimage_submodule(src, zero_submodule(dst), f)
        let ker = linear_map_kernel_submodule(src, dst, f)
        linear_map_preimage_submodule_carrier_eq(src, zero_submodule(dst), f)
        linear_map_kernel_submodule_carrier_eq(src, dst, f)
        pre.carrier = src
        ker.carrier = src
        pre.carrier = ker.carrier
        forall(x: M) {
            linear_map_preimage_submodule_zero_eq_kernel_at(src, dst, f, x)
            pre.contains(x) = ker.contains(x)
        }
        pre.carrier = ker.carrier and forall(x: M) { pre.contains(x) = ker.contains(x) }
        submodule_eq_of_carrier_contains_at_eq(pre, ker)
        pre = ker
        linear_map_preimage_submodule(src, zero_submodule(dst), f) =
            linear_map_kernel_submodule(src, dst, f)
    }
}

/// The preimage of the full submodule under a linear map is the full source submodule.
theorem linear_map_preimage_submodule_full_eq[R: Ring, M: AddCommGroup, N: AddCommGroup](
    src: Module[R, M], dst: Module[R, N], f: M -> N
) {
    is_linear_map(src, dst, f) implies
        linear_map_preimage_submodule(src, full_submodule(dst), f) = full_submodule(src)
} by {
    if is_linear_map(src, dst, f) {
        let pre = linear_map_preimage_submodule(src, full_submodule(dst), f)
        let full = full_submodule(src)
        linear_map_preimage_submodule_carrier_eq(src, full_submodule(dst), f)
        full_submodule_carrier_eq(src)
        pre.carrier = src
        full.carrier = src
        pre.carrier = full.carrier
        forall(x: M) {
            linear_map_preimage_submodule_full_contains_at(src, dst, f, x)
            pre.contains(x) = full.contains(x)
        }
        pre.carrier = full.carrier and forall(x: M) { pre.contains(x) = full.contains(x) }
        submodule_eq_of_carrier_contains_at_eq(pre, full)
        pre = full
        linear_map_preimage_submodule(src, full_submodule(dst), f) = full_submodule(src)
    }
}

/// The kernel of a composed linear map is the preimage of the outer kernel along the inner map.
theorem linear_map_kernel_compose_eq_preimage_kernel[R: Ring, M: AddCommGroup, N: AddCommGroup, K: AddCommGroup](
    src: Module[R, M], mid: Module[R, N], dst: Module[R, K], f: N -> K, g: M -> N
) {
    is_linear_map(mid, dst, f) and is_linear_map(src, mid, g) implies
        linear_map_kernel_submodule(src, dst, compose(f, g)) =
            linear_map_preimage_submodule(src, linear_map_kernel_submodule(mid, dst, f), g)
} by {
    if is_linear_map(mid, dst, f) and is_linear_map(src, mid, g) {
        let lhs = linear_map_kernel_submodule(src, dst, compose(f, g))
        let rhs = linear_map_preimage_submodule(src, linear_map_kernel_submodule(mid, dst, f), g)
        compose_is_linear_map(src, mid, dst, f, g)
        is_linear_map(src, dst, compose(f, g))
        linear_map_kernel_submodule_carrier_eq(src, dst, compose(f, g))
        lhs.carrier = src
        linear_map_preimage_submodule_carrier_eq(src, linear_map_kernel_submodule(mid, dst, f), g)
        rhs.carrier = src
        lhs.carrier = rhs.carrier
        linear_map_kernel_submodule_carrier_eq(mid, dst, f)
        linear_map_kernel_submodule(mid, dst, f).carrier = mid
        forall(x: M) {
            linear_map_preimage_submodule_contains_eq(src, linear_map_kernel_submodule(mid, dst, f), g, x)
            rhs.contains(x) = linear_map_kernel_submodule(mid, dst, f).contains(g(x))
            if lhs.contains(x) {
                linear_map_kernel_submodule_map_eq_zero_of_contains(src, dst, compose(f, g), x)
                compose(f, g)(x) = K.0
                f(g(x)) = K.0
                linear_map_kernel_submodule_contains_of_map_eq_zero(mid, dst, f, g(x))
                linear_map_kernel_submodule(mid, dst, f).contains(g(x))
                rhs.contains(x)
            }
            if rhs.contains(x) {
                linear_map_kernel_submodule(mid, dst, f).contains(g(x))
                linear_map_kernel_submodule_map_eq_zero_of_contains(mid, dst, f, g(x))
                f(g(x)) = K.0
                compose(f, g)(x) = K.0
                linear_map_kernel_submodule_contains_of_map_eq_zero(src, dst, compose(f, g), x)
                lhs.contains(x)
            }
            lhs.contains(x) = rhs.contains(x)
        }
        lhs.carrier = rhs.carrier and forall(x: M) { lhs.contains(x) = rhs.contains(x) }
        submodule_eq_of_carrier_contains_at_eq(lhs, rhs)
        lhs = rhs
        linear_map_kernel_submodule(src, dst, compose(f, g)) =
            linear_map_preimage_submodule(src, linear_map_kernel_submodule(mid, dst, f), g)
    }
}
