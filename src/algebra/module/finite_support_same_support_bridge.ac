from finite_set import FiniteSet
from algebra.module.finite_support import has_support_in, has_support_in_apply_zero, is_finitely_supported
from algebra.zero import Zero

/// Functions with identical zero loci.
define same_zero_support[T, A: Zero](f: T -> A, g: T -> A) -> Bool {
    forall(t: T) {
        (f(t) = A.0) = (g(t) = A.0)
    }
}

/// Equal zero loci transfer a displayed finite support.
theorem has_support_in_of_same_zero_support[T, A: Zero](f: T -> A, g: T -> A, s: FiniteSet[T]) {
    same_zero_support(f, g) and has_support_in(f, s) implies has_support_in(g, s)
} by {
    same_zero_support(f, g)
    has_support_in(f, s)
    forall(t: T) {
        if not s.contains(t) {
            has_support_in_apply_zero[T, A](f, s, t)
            f(t) = A.0
            same_zero_support(f, g) = forall(x: T) {
                (f(x) = A.0) = (g(x) = A.0)
            }
            (f(t) = A.0) = (g(t) = A.0)
            g(t) = A.0
        }
    }
}

/// Equal zero loci make displayed finite-support membership equivalent.
theorem has_support_in_iff_same_zero_support[T, A: Zero](f: T -> A, g: T -> A, s: FiniteSet[T]) {
    same_zero_support(f, g) implies (has_support_in(f, s) = has_support_in(g, s))
} by {
    if same_zero_support(f, g) {
        if has_support_in(f, s) {
            has_support_in_of_same_zero_support[T, A](f, g, s)
            has_support_in(g, s)
        }
        if has_support_in(g, s) {
            forall(t: T) {
                if not s.contains(t) {
                    has_support_in_apply_zero[T, A](g, s, t)
                    g(t) = A.0
                    same_zero_support(f, g) = forall(x: T) {
                        (f(x) = A.0) = (g(x) = A.0)
                    }
                    (f(t) = A.0) = (g(t) = A.0)
                    f(t) = A.0
                }
            }
            has_support_in(f, s)
        }
        has_support_in(f, s) = has_support_in(g, s)
    }
}

/// Equal zero loci make finite support equivalent.
theorem is_finitely_supported_iff_same_zero_support[T, A: Zero](f: T -> A, g: T -> A) {
    same_zero_support(f, g) implies (is_finitely_supported(f) = is_finitely_supported(g))
} by {
    if same_zero_support(f, g) {
        if is_finitely_supported(f) {
            let s: FiniteSet[T] satisfy {
                has_support_in(f, s)
            }
            has_support_in_of_same_zero_support[T, A](f, g, s)
            exists(u: FiniteSet[T]) {
                u = s and has_support_in(g, u)
            }
            is_finitely_supported(g)
        }
        if is_finitely_supported(g) {
            let s: FiniteSet[T] satisfy {
                has_support_in(g, s)
            }
            has_support_in_iff_same_zero_support[T, A](f, g, s)
            has_support_in(f, s)
            exists(u: FiniteSet[T]) {
                u = s and has_support_in(f, u)
            }
            is_finitely_supported(f)
        }
        is_finitely_supported(f) = is_finitely_supported(g)
    }
}
