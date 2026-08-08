/// Bundled `ModuleHom` accessors for the kernel-quotient first-isomorphism infrastructure.
/// The underlying quotient construction is still the unbundled linear-map quotient;
/// this module lets bundled-homomorphism users state the canonical projection,
/// induced map, equality criterion, and kernel/image bridges without manually
/// unpacking `.src`, `.dst`, and `.hom` at every use.

from algebra.ring.ring import Ring
from algebra.add_comm_group import AddCommGroup
from algebra.module.module_hom import ModuleHom, module_hom_is_linear_map
from data.basic.equivalence import QuotientOver, kernel_relation
from data.basic.quotient_algebra import linear_map_kernel_quotient_project,
    linear_map_kernel_quotient_induced,
    linear_map_kernel_quotient_induced_project,
    linear_map_kernel_quotient_induced_injective,
    linear_map_kernel_quotient_zero,
    linear_map_kernel_quotient_project_zero
from data.basic.quotient_algebra_first_iso import linear_map_kernel_quotient_image_eq_iff_project_eq,
    linear_map_kernel_quotient_project_eq_iff_kernel_relation,
    linear_map_kernel_quotient_induced_eq_iff_kernel_relation
from algebra.module.linear_map_quotient_hom import linear_map_kernel_quotient_project_eq_zero_iff_image_eq_zero,
    linear_map_kernel_quotient_induced_surjective_onto_image
from algebra.module.submodule import module_hom_kernel, module_hom_kernel_contains_eq

/// The canonical projection from a bundled module homomorphism's source to its kernel quotient.
define module_hom_kernel_quotient_project[R: Ring, M: AddCommGroup, N: AddCommGroup](
    f: ModuleHom[R, M, N], a: M
) -> QuotientOver[M] {
    linear_map_kernel_quotient_project(f.src, f.dst, f.hom, a)
}

/// The zero element of the kernel quotient of a bundled module homomorphism.
define module_hom_kernel_quotient_zero[R: Ring, M: AddCommGroup, N: AddCommGroup](
    f: ModuleHom[R, M, N]
) -> QuotientOver[M] {
    linear_map_kernel_quotient_zero(f.src, f.dst, f.hom)
}

/// The map from the kernel quotient induced by a bundled module homomorphism.
define module_hom_kernel_quotient_induced[R: Ring, M: AddCommGroup, N: AddCommGroup](
    f: ModuleHom[R, M, N]
) -> QuotientOver[M] -> N {
    linear_map_kernel_quotient_induced(f.src, f.dst, f.hom)
}

/// The canonical projection sends source zero to quotient zero.
theorem module_hom_kernel_quotient_project_zero[R: Ring, M: AddCommGroup, N: AddCommGroup](
    f: ModuleHom[R, M, N]
) {
    module_hom_kernel_quotient_project(f, M.0) = module_hom_kernel_quotient_zero(f)
} by {
    linear_map_kernel_quotient_project_zero(f.src, f.dst, f.hom)
}

/// The induced map agrees with the bundled homomorphism on canonical projections.
theorem module_hom_kernel_quotient_induced_project[R: Ring, M: AddCommGroup, N: AddCommGroup](
    f: ModuleHom[R, M, N], a: M
) {
    module_hom_kernel_quotient_induced(f, module_hom_kernel_quotient_project(f, a)) = f.hom(a)
} by {
    linear_map_kernel_quotient_induced_project(f.src, f.dst, f.hom, a)
}

/// Induced values on projected representatives are equal exactly when the projections are equal.
theorem module_hom_kernel_quotient_induced_eq_iff_project_eq[R: Ring, M: AddCommGroup, N: AddCommGroup](
    f: ModuleHom[R, M, N], a: M, b: M
) {
    (module_hom_kernel_quotient_induced(f, module_hom_kernel_quotient_project(f, a)) =
        module_hom_kernel_quotient_induced(f, module_hom_kernel_quotient_project(f, b))) =
        (module_hom_kernel_quotient_project(f, a) = module_hom_kernel_quotient_project(f, b))
} by {
    if module_hom_kernel_quotient_induced(f, module_hom_kernel_quotient_project(f, a)) =
        module_hom_kernel_quotient_induced(f, module_hom_kernel_quotient_project(f, b)) {
        linear_map_kernel_quotient_induced_injective(f.src, f.dst, f.hom, a, b)
        linear_map_kernel_quotient_project(f.src, f.dst, f.hom, a) =
            linear_map_kernel_quotient_project(f.src, f.dst, f.hom, b)
        module_hom_kernel_quotient_project(f, a) = module_hom_kernel_quotient_project(f, b)
    }
}

/// Two source elements have equal bundled-hom images exactly when their kernel-quotient projections agree.
theorem module_hom_kernel_quotient_image_eq_iff_project_eq[R: Ring, M: AddCommGroup, N: AddCommGroup](
    f: ModuleHom[R, M, N], a: M, b: M
) {
    (f.hom(a) = f.hom(b)) =
        (module_hom_kernel_quotient_project(f, a) = module_hom_kernel_quotient_project(f, b))
} by {
    linear_map_kernel_quotient_image_eq_iff_project_eq(f.src, f.dst, f.hom, a, b)
}

/// Equality of bundled-hom kernel-quotient projections is the kernel relation of the underlying map.
theorem module_hom_kernel_quotient_project_eq_iff_kernel_relation[R: Ring, M: AddCommGroup, N: AddCommGroup](
    f: ModuleHom[R, M, N], a: M, b: M
) {
    (module_hom_kernel_quotient_project(f, a) = module_hom_kernel_quotient_project(f, b)) =
        kernel_relation(f.hom, a, b)
} by {
    linear_map_kernel_quotient_project_eq_iff_kernel_relation(f.src, f.dst, f.hom, a, b)
}

/// Induced values on bundled-hom quotient projections are equal exactly on kernel-related representatives.
theorem module_hom_kernel_quotient_induced_eq_iff_kernel_relation[R: Ring, M: AddCommGroup, N: AddCommGroup](
    f: ModuleHom[R, M, N], a: M, b: M
) {
    (module_hom_kernel_quotient_induced(f, module_hom_kernel_quotient_project(f, a)) =
        module_hom_kernel_quotient_induced(f, module_hom_kernel_quotient_project(f, b))) =
        kernel_relation(f.hom, a, b)
} by {
    linear_map_kernel_quotient_induced_eq_iff_kernel_relation(f.src, f.dst, f.hom, a, b)
}

/// A projected representative is quotient-zero exactly when its bundled-hom image is zero.
theorem module_hom_kernel_quotient_project_eq_zero_iff_map_eq_zero[R: Ring, M: AddCommGroup, N: AddCommGroup](
    f: ModuleHom[R, M, N], a: M
) {
    (module_hom_kernel_quotient_project(f, a) = module_hom_kernel_quotient_zero(f)) =
        (f.hom(a) = N.0)
} by {
    module_hom_is_linear_map(f)
    linear_map_kernel_quotient_project_eq_zero_iff_image_eq_zero(f.src, f.dst, f.hom, a)
}

/// A projected representative is quotient-zero exactly when the representative belongs to the bundled kernel submodule.
theorem module_hom_kernel_quotient_project_eq_zero_iff_kernel_contains[R: Ring, M: AddCommGroup, N: AddCommGroup](
    f: ModuleHom[R, M, N], a: M
) {
    (module_hom_kernel_quotient_project(f, a) = module_hom_kernel_quotient_zero(f)) =
        module_hom_kernel(f).contains(a)
} by {
    module_hom_kernel_quotient_project_eq_zero_iff_map_eq_zero(f, a)
    module_hom_kernel_contains_eq(f, a)
}

/// Every bundled-hom image value is attained by the induced map from the kernel quotient.
theorem module_hom_kernel_quotient_induced_surjective_onto_image[R: Ring, M: AddCommGroup, N: AddCommGroup](
    f: ModuleHom[R, M, N], a: M
) {
    exists(q: QuotientOver[M]) {
        module_hom_kernel_quotient_induced(f, q) = f.hom(a)
    }
} by {
    linear_map_kernel_quotient_induced_surjective_onto_image(f.src, f.dst, f.hom, a)
}
