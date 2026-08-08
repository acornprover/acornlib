from algebra.ring.ring import Ring
from algebra.add_comm_group import AddCommGroup
from algebra.add_comm_group import AddCommGroupHom, is_add_comm_group_hom,
    add_comm_group_hom_zero, add_comm_group_hom_neg
from algebra.add_subgroup import AddSubgroup, add_zero_constraint, add_closure_constraint,
    neg_constraint, add_subgroup_constraint, add_subgroup_rel, add_subgroup_quotient_relation,
    add_subgroup_quotient_relation_rel, add_subgroup_rel_is_equivalence,
    add_subgroup_quotient_relation_respects_add, add_subgroup_quotient_relation_respects_neg,
    add_subgroup_quotient_add, add_subgroup_quotient_neg, add_subgroup_quotient_zero,
    add_subgroup_quotient_add_projection, add_subgroup_quotient_neg_projection
from algebra.add_submonoid import AddSubmonoid, add_submonoid_zero_constraint,
    add_submonoid_closure_constraint, add_submonoid_constraint
from algebra.module.module import Module, module_smul_zero_right, module_smul_add_right, module_smul_neg_right,
    module_smul_add_left, module_smul_assoc, module_smul_one
from algebra.module.module_hom import is_linear_map, preserves_add, preserves_smul, linear_map_add, linear_map_smul,
    is_linear_equiv, linear_equiv_is_linear_map, linear_equiv_is_surjective,
    linear_equiv_apply_eq_zero_iff_eq_zero, ModuleHom, module_hom_is_linear_map, module_hom_sub,
    module_hom_zero, module_hom_identity, module_hom_identity_hom_at,
    module_hom_trivial, module_hom_trivial_hom_at
from data.basic.equivalence import QuotientRelation, QuotientOver, quotient_over_mk,
    quotient_over_unary_op, quotient_over_unary_op_projection,
    quotient_over_unary_op_projection_compatible, quotient_unary_respects,
    quotient_unary_respects_eq_respects_unary_op, quotient_relation_new_round_trip,
    quotient_over_mk_eq_iff_rel, quotient_over_mk_eq_of_rel, rel_of_quotient_over_mk_eq
from data.basic.relation_transport import predicate_subset, predicate_subset_step, respects_unary_op
from data.basic.set import Set, singleton_contains_eq, singleton_set_is_finite, empty_set_is_finite,
    union_contains_eq, union_contains_left, union_contains_right,
    is_subset_monotone_map, is_set_extensive_map, is_set_idempotent_map,
    is_set_closure_operator
from data.basic.functions import predicate_extensionality, is_injective_fn, is_surjective_fn

/// True if a subset of a module contains the additive identity.
define submodule_zero_constraint[R: Ring, M: AddCommGroup](
    carrier: Module[R, M], contains: M -> Bool
) -> Bool {
    contains(M.0)
}

/// True if a subset of a module is closed under addition.
define submodule_add_constraint[R: Ring, M: AddCommGroup](
    carrier: Module[R, M], contains: M -> Bool
) -> Bool {
    forall(a: M, b: M) {
        contains(a) and contains(b) implies contains(a + b)
    }
}

/// True if a subset of a module is closed under additive negation.
define submodule_neg_constraint[R: Ring, M: AddCommGroup](
    carrier: Module[R, M], contains: M -> Bool
) -> Bool {
    forall(a: M) {
        contains(a) implies contains(-a)
    }
}

/// True if a subset of a module is closed under scalar multiplication.
define submodule_smul_constraint[R: Ring, M: AddCommGroup](
    carrier: Module[R, M], contains: M -> Bool
) -> Bool {
    forall(r: R, a: M) {
        contains(a) implies contains(carrier.smul(r, a))
    }
}

/// True if a subset is a submodule of the given module.
define is_submodule[R: Ring, M: AddCommGroup](
    carrier: Module[R, M], contains: M -> Bool
) -> Bool {
    submodule_zero_constraint(carrier, contains)
    and submodule_add_constraint(carrier, contains)
    and submodule_neg_constraint(carrier, contains)
    and submodule_smul_constraint(carrier, contains)
}

/// A submodule of a module, represented as a subset closed under the module operations.
structure Submodule[R: Ring, M: AddCommGroup] {
    /// The ambient module.
    carrier: Module[R, M]

    /// True if the given element is a member of this submodule.
    contains: M -> Bool
} constraint {
    is_submodule(carrier, contains)
}

/// Submodule extensionality: two submodules of the same module are equal when they have the same elements.
theorem submodule_ext[R: Ring, M: AddCommGroup](
    a: Submodule[R, M], b: Submodule[R, M]
) {
    a.carrier = b.carrier and (forall(x: M) { a.contains(x) = b.contains(x) }) implies a = b
} by {
    if a.carrier = b.carrier and forall(x: M) { a.contains(x) = b.contains(x) } {
        a.contains = b.contains
    }
}

/// Equal submodules have equal ambient modules.
theorem submodule_eq_carrier[R: Ring, M: AddCommGroup](
    a: Submodule[R, M], b: Submodule[R, M]
) {
    a = b implies a.carrier = b.carrier
}

/// Equal submodules have equal membership predicates.
theorem submodule_eq_contains[R: Ring, M: AddCommGroup](
    a: Submodule[R, M], b: Submodule[R, M]
) {
    a = b implies a.contains = b.contains
}

/// Equal submodules have equal membership at every element.
theorem submodule_eq_contains_at[R: Ring, M: AddCommGroup](
    a: Submodule[R, M], b: Submodule[R, M], x: M
) {
    a = b implies a.contains(x) = b.contains(x)
} by {
    if a = b {
        a.contains = b.contains
        a.contains(x) = b.contains(x)
    }
}

/// Equality of submodules transports predicates on submodules.
theorem submodule_eq_transport_predicate[R: Ring, M: AddCommGroup](
    p: Submodule[R, M] -> Bool,
    a: Submodule[R, M],
    b: Submodule[R, M]
) {
    a = b and p(a) implies p(b)
}

/// Equality of submodules transports predicates on submodules in the reverse direction.
theorem submodule_eq_transport_predicate_rev[R: Ring, M: AddCommGroup](
    p: Submodule[R, M] -> Bool,
    a: Submodule[R, M],
    b: Submodule[R, M]
) {
    a = b and p(b) implies p(a)
}

/// Equality of ambient modules and membership predicates determines equality of submodules.
theorem submodule_eq_of_carrier_contains_eq[R: Ring, M: AddCommGroup](
    a: Submodule[R, M], b: Submodule[R, M]
) {
    a.carrier = b.carrier and a.contains = b.contains implies a = b
} by {
    if a.carrier = b.carrier and a.contains = b.contains {
        forall(x: M) {
            a.contains(x) = b.contains(x)
        }
        submodule_ext(a, b)
    }
}

/// Equality of ambient modules and pointwise membership determines equality of submodules.
theorem submodule_eq_of_carrier_contains_at_eq[R: Ring, M: AddCommGroup](
    a: Submodule[R, M], b: Submodule[R, M]
) {
    a.carrier = b.carrier and (forall(x: M) { a.contains(x) = b.contains(x) }) implies a = b
} by {
    if a.carrier = b.carrier and forall(x: M) { a.contains(x) = b.contains(x) } {
        submodule_ext(a, b)
    }
}

/// A submodule contains the additive identity.
theorem submodule_contains_zero[R: Ring, M: AddCommGroup](s: Submodule[R, M]) {
    s.contains(M.0)
} by {
    is_submodule(s.carrier, s.contains)
    is_submodule(s.carrier, s.contains) =
        (submodule_zero_constraint(s.carrier, s.contains)
         and submodule_add_constraint(s.carrier, s.contains)
         and submodule_neg_constraint(s.carrier, s.contains)
         and submodule_smul_constraint(s.carrier, s.contains))
    submodule_zero_constraint(s.carrier, s.contains)
    submodule_zero_constraint(s.carrier, s.contains) = s.contains(M.0)
}

/// A submodule is closed under addition.
theorem submodule_contains_add[R: Ring, M: AddCommGroup](s: Submodule[R, M], a: M, b: M) {
    s.contains(a) and s.contains(b) implies s.contains(a + b)
} by {
    if s.contains(a) and s.contains(b) {
        is_submodule(s.carrier, s.contains)
        is_submodule(s.carrier, s.contains) =
            (submodule_zero_constraint(s.carrier, s.contains)
             and submodule_add_constraint(s.carrier, s.contains)
             and submodule_neg_constraint(s.carrier, s.contains)
             and submodule_smul_constraint(s.carrier, s.contains))
        submodule_add_constraint(s.carrier, s.contains)
        submodule_add_constraint(s.carrier, s.contains) = forall(x: M, y: M) {
            s.contains(x) and s.contains(y) implies s.contains(x + y)
        }
        s.contains(a + b)
    }
}

/// A submodule is closed under additive negation.
theorem submodule_contains_neg[R: Ring, M: AddCommGroup](s: Submodule[R, M], a: M) {
    s.contains(a) implies s.contains(-a)
} by {
    if s.contains(a) {
        is_submodule(s.carrier, s.contains)
        is_submodule(s.carrier, s.contains) =
            (submodule_zero_constraint(s.carrier, s.contains)
             and submodule_add_constraint(s.carrier, s.contains)
             and submodule_neg_constraint(s.carrier, s.contains)
             and submodule_smul_constraint(s.carrier, s.contains))
        submodule_neg_constraint(s.carrier, s.contains)
        submodule_neg_constraint(s.carrier, s.contains) = forall(x: M) {
            s.contains(x) implies s.contains(-x)
        }
        s.contains(-a)
    }
}

/// A submodule is closed under scalar multiplication.
theorem submodule_contains_smul[R: Ring, M: AddCommGroup](s: Submodule[R, M], r: R, a: M) {
    s.contains(a) implies s.contains(s.carrier.smul(r, a))
} by {
    if s.contains(a) {
        is_submodule(s.carrier, s.contains)
        is_submodule(s.carrier, s.contains) =
            (submodule_zero_constraint(s.carrier, s.contains)
             and submodule_add_constraint(s.carrier, s.contains)
             and submodule_neg_constraint(s.carrier, s.contains)
             and submodule_smul_constraint(s.carrier, s.contains))
        submodule_smul_constraint(s.carrier, s.contains)
        submodule_smul_constraint(s.carrier, s.contains) = forall(q: R, x: M) {
            s.contains(x) implies s.contains(s.carrier.smul(q, x))
        }
        s.contains(s.carrier.smul(r, a))
    }
}

/// Membership is unchanged by additive negation.
theorem submodule_contains_neg_iff[R: Ring, M: AddCommGroup](s: Submodule[R, M], a: M) {
    s.contains(-a) = s.contains(a)
} by {
    if s.contains(-a) {
        submodule_contains_neg(s, -a)
        s.contains(--a)
        --a = a
        s.contains(a)
    }
    if s.contains(a) {
        submodule_contains_neg(s, a)
        s.contains(-a)
    }
    s.contains(-a) = s.contains(a)
}

/// A submodule is closed under subtraction.
theorem submodule_contains_sub[R: Ring, M: AddCommGroup](s: Submodule[R, M], a: M, b: M) {
    s.contains(a) and s.contains(b) implies s.contains(a - b)
} by {
    if s.contains(a) and s.contains(b) {
        submodule_contains_neg(s, b)
        s.contains(-b)
        submodule_contains_add(s, a, -b)
        s.contains(a + -b)
        a - b = a + -b
    }
}

/// A submodule is closed under sums of three elements.
theorem submodule_contains_add_add[R: Ring, M: AddCommGroup](s: Submodule[R, M], a: M, b: M, c: M) {
    s.contains(a) and s.contains(b) and s.contains(c) implies s.contains(a + b + c)
} by {
    if s.contains(a) and s.contains(b) and s.contains(c) {
        submodule_contains_add(s, a, b)
        s.contains(a + b)
        submodule_contains_add(s, a + b, c)
        s.contains(a + b + c)
    }
}

/// A submodule is closed under a negated element added on the left.
theorem submodule_contains_neg_add[R: Ring, M: AddCommGroup](s: Submodule[R, M], a: M, b: M) {
    s.contains(a) and s.contains(b) implies s.contains(-a + b)
} by {
    if s.contains(a) and s.contains(b) {
        submodule_contains_neg(s, a)
        s.contains(-a)
        submodule_contains_add(s, -a, b)
        s.contains(-a + b)
    }
}

/// The common membership predicate of two submodules of the same module.
define submodule_intersection_contains[R: Ring, M: AddCommGroup](
    a: Submodule[R, M], b: Submodule[R, M], x: M
) -> Bool {
    a.contains(x) and b.contains(x)
}

/// Membership in the common part means membership in both submodules.
theorem submodule_intersection_contains_eq[R: Ring, M: AddCommGroup](
    a: Submodule[R, M], b: Submodule[R, M], x: M
) {
    submodule_intersection_contains(a, b, x) = (a.contains(x) and b.contains(x))
} by {
    submodule_intersection_contains(a, b, x) = (a.contains(x) and b.contains(x))
}

/// The common part is contained in the left submodule.
theorem submodule_intersection_subset_left[R: Ring, M: AddCommGroup](
    a: Submodule[R, M], b: Submodule[R, M], x: M
) {
    submodule_intersection_contains(a, b, x) implies a.contains(x)
} by {
    if submodule_intersection_contains(a, b, x) {
        submodule_intersection_contains_eq(a, b, x)
        a.contains(x)
    }
}

/// The common part is contained in the right submodule.
theorem submodule_intersection_subset_right[R: Ring, M: AddCommGroup](
    a: Submodule[R, M], b: Submodule[R, M], x: M
) {
    submodule_intersection_contains(a, b, x) implies b.contains(x)
} by {
    if submodule_intersection_contains(a, b, x) {
        submodule_intersection_contains_eq(a, b, x)
        b.contains(x)
    }
}

/// The common part of two submodules of the same module contains the additive identity.
theorem submodule_intersection_zero_constraint[R: Ring, M: AddCommGroup](
    a: Submodule[R, M], b: Submodule[R, M]
) {
    submodule_zero_constraint(a.carrier, submodule_intersection_contains(a, b))
} by {
    submodule_contains_zero(a)
    submodule_contains_zero(b)
    submodule_intersection_contains(a, b, M.0)
}

/// The common part of two submodules of the same module is closed under addition.
theorem submodule_intersection_add_constraint[R: Ring, M: AddCommGroup](
    a: Submodule[R, M], b: Submodule[R, M]
) {
    submodule_add_constraint(a.carrier, submodule_intersection_contains(a, b))
} by {
    forall(x: M, y: M) {
        if submodule_intersection_contains(a, b, x) and submodule_intersection_contains(a, b, y) {
            a.contains(x)
            b.contains(x)
            a.contains(y)
            b.contains(y)
            submodule_contains_add(a, x, y)
            submodule_contains_add(b, x, y)
            a.contains(x + y)
            b.contains(x + y)
            submodule_intersection_contains(a, b, x + y)
        }
    }
}

/// The common part of two submodules of the same module is closed under additive negation.
theorem submodule_intersection_neg_constraint[R: Ring, M: AddCommGroup](
    a: Submodule[R, M], b: Submodule[R, M]
) {
    submodule_neg_constraint(a.carrier, submodule_intersection_contains(a, b))
} by {
    forall(x: M) {
        if submodule_intersection_contains(a, b, x) {
            a.contains(x)
            b.contains(x)
            submodule_contains_neg(a, x)
            submodule_contains_neg(b, x)
            a.contains(-x)
            b.contains(-x)
            submodule_intersection_contains(a, b, -x)
        }
    }
}

/// The common part of two submodules of the same module is closed under scalar multiplication.
theorem submodule_intersection_smul_constraint[R: Ring, M: AddCommGroup](
    a: Submodule[R, M], b: Submodule[R, M]
) {
    a.carrier = b.carrier implies submodule_smul_constraint(a.carrier, submodule_intersection_contains(a, b))
} by {
    if a.carrier = b.carrier {
        forall(r: R, x: M) {
            if submodule_intersection_contains(a, b, x) {
                a.contains(x)
                b.contains(x)
                submodule_contains_smul(a, r, x)
                submodule_contains_smul(b, r, x)
                a.carrier.smul(r, x) = b.carrier.smul(r, x)
                a.contains(a.carrier.smul(r, x))
                b.contains(a.carrier.smul(r, x))
                submodule_intersection_contains(a, b, a.carrier.smul(r, x))
            }
        }
    }
}

/// The common part of two submodules of the same module is a submodule.
theorem submodule_intersection_is_submodule[R: Ring, M: AddCommGroup](
    a: Submodule[R, M], b: Submodule[R, M]
) {
    a.carrier = b.carrier implies is_submodule(a.carrier, submodule_intersection_contains(a, b))
} by {
    if a.carrier = b.carrier {
        let p = submodule_intersection_contains(a, b)
        is_submodule(a.carrier, p) =
            (submodule_zero_constraint(a.carrier, p)
             and submodule_add_constraint(a.carrier, p)
             and submodule_neg_constraint(a.carrier, p)
             and submodule_smul_constraint(a.carrier, p))
        submodule_intersection_zero_constraint(a, b)
        submodule_intersection_add_constraint(a, b)
        submodule_intersection_neg_constraint(a, b)
        submodule_intersection_smul_constraint(a, b)
    }
}

/// The bundled common part of two submodules, using the left submodule when the ambient modules differ.
define submodule_bundled_intersection_contains[R: Ring, M: AddCommGroup](
    a: Submodule[R, M], b: Submodule[R, M], x: M
) -> Bool {
    if a.carrier = b.carrier {
        submodule_intersection_contains(a, b, x)
    } else {
        a.contains(x)
    }
}

/// The bundled common part agrees with common membership when the ambient modules are equal.
theorem submodule_bundled_intersection_contains_eq_of_carrier_eq[R: Ring, M: AddCommGroup](
    a: Submodule[R, M], b: Submodule[R, M], x: M
) {
    a.carrier = b.carrier implies
        submodule_bundled_intersection_contains(a, b, x) = submodule_intersection_contains(a, b, x)
} by {
    if a.carrier = b.carrier {
        submodule_bundled_intersection_contains(a, b, x) = submodule_intersection_contains(a, b, x)
    }
}

/// The bundled common part agrees with the left submodule when the ambient modules differ.
theorem submodule_bundled_intersection_contains_eq_left_of_carrier_ne[R: Ring, M: AddCommGroup](
    a: Submodule[R, M], b: Submodule[R, M], x: M
) {
    a.carrier != b.carrier implies
        submodule_bundled_intersection_contains(a, b, x) = a.contains(x)
} by {
    if a.carrier != b.carrier {
        submodule_bundled_intersection_contains(a, b, x) = a.contains(x)
    }
}

/// The bundled common part is a submodule of the left ambient module.
theorem submodule_bundled_intersection_is_submodule[R: Ring, M: AddCommGroup](
    a: Submodule[R, M], b: Submodule[R, M]
) {
    is_submodule(a.carrier, submodule_bundled_intersection_contains(a, b))
} by {
    if a.carrier = b.carrier {
        submodule_intersection_is_submodule(a, b)
        is_submodule(a.carrier, submodule_intersection_contains(a, b))
        forall(x: M) {
            submodule_bundled_intersection_contains_eq_of_carrier_eq(a, b, x)
            submodule_bundled_intersection_contains(a, b, x) = submodule_intersection_contains(a, b, x)
        }
        predicate_extensionality(
            submodule_bundled_intersection_contains(a, b),
            submodule_intersection_contains(a, b)
        )
        submodule_bundled_intersection_contains(a, b) = submodule_intersection_contains(a, b)
        is_submodule(a.carrier, submodule_bundled_intersection_contains(a, b))
    } else {
        forall(x: M) {
            submodule_bundled_intersection_contains_eq_left_of_carrier_ne(a, b, x)
            submodule_bundled_intersection_contains(a, b, x) = a.contains(x)
        }
        predicate_extensionality(submodule_bundled_intersection_contains(a, b), a.contains)
        submodule_bundled_intersection_contains(a, b) = a.contains
        is_submodule(a.carrier, a.contains)
        is_submodule(a.carrier, submodule_bundled_intersection_contains(a, b))
    }
}

/// The bundled common part of two submodules.
let submodule_intersection[R: Ring, M: AddCommGroup](a: Submodule[R, M], b: Submodule[R, M]) -> result: Submodule[R, M] satisfy {
    Submodule.new(a.carrier, submodule_bundled_intersection_contains(a, b)) = Option.some(result)
} by {
    submodule_bundled_intersection_is_submodule(a, b)
}

/// The bundled common part has the left ambient module.
theorem submodule_intersection_carrier_eq[R: Ring, M: AddCommGroup](a: Submodule[R, M], b: Submodule[R, M]) {
    submodule_intersection(a, b).carrier = a.carrier
}

/// Common membership is symmetric.
theorem submodule_intersection_contains_comm_at[R: Ring, M: AddCommGroup](
    a: Submodule[R, M], b: Submodule[R, M], x: M
) {
    submodule_intersection_contains(a, b, x) = submodule_intersection_contains(b, a, x)
} by {
    submodule_intersection_contains_eq(a, b, x)
    submodule_intersection_contains_eq(b, a, x)
    submodule_intersection_contains(a, b, x) = submodule_intersection_contains(b, a, x)
}

/// Common membership is associative from left to right.
theorem submodule_intersection_contains_assoc_left[R: Ring, M: AddCommGroup](
    a: Submodule[R, M], b: Submodule[R, M], c: Submodule[R, M], x: M
) {
    submodule_intersection_contains(a, b, x) and c.contains(x)
        implies a.contains(x) and submodule_intersection_contains(b, c, x)
} by {
    if submodule_intersection_contains(a, b, x) and c.contains(x) {
        submodule_intersection_contains_eq(a, b, x)
        a.contains(x)
        b.contains(x)
        submodule_intersection_contains_eq(b, c, x)
        submodule_intersection_contains(b, c, x)
        a.contains(x) and submodule_intersection_contains(b, c, x)
    }
}

/// Common membership is associative from right to left.
theorem submodule_intersection_contains_assoc_right[R: Ring, M: AddCommGroup](
    a: Submodule[R, M], b: Submodule[R, M], c: Submodule[R, M], x: M
) {
    a.contains(x) and submodule_intersection_contains(b, c, x)
        implies submodule_intersection_contains(a, b, x) and c.contains(x)
} by {
    if a.contains(x) and submodule_intersection_contains(b, c, x) {
        submodule_intersection_contains_eq(b, c, x)
        b.contains(x)
        c.contains(x)
        submodule_intersection_contains_eq(a, b, x)
        submodule_intersection_contains(a, b, x)
        submodule_intersection_contains(a, b, x) and c.contains(x)
    }
}

/// Common membership with itself is ordinary membership.
theorem submodule_intersection_contains_idempotent_at[R: Ring, M: AddCommGroup](
    a: Submodule[R, M], x: M
) {
    submodule_intersection_contains(a, a, x) = a.contains(x)
} by {
    submodule_intersection_contains_eq(a, a, x)
    submodule_intersection_contains(a, a, x) = a.contains(x)
}

attributes Submodule[R: Ring, M: AddCommGroup] {
    /// Submodule extensionality from equality of ambient modules and pointwise equality of membership.
    let ext = submodule_ext[R, M]

    /// The subset of module elements belonging to this submodule.
    define as_set(self) -> Set[M] {
        Set[M].new(self.contains)
    }

    /// The common part of this submodule and another submodule.
    let intersection: (Submodule[R, M], Submodule[R, M]) -> Submodule[R, M] = submodule_intersection
}

/// Membership in the underlying set is membership in the submodule.
theorem submodule_as_set_contains_eq[R: Ring, M: AddCommGroup](s: Submodule[R, M], x: M) {
    s.as_set.contains(x) = s.contains(x)
} by {
    s.as_set.contains(x) = s.contains(x)
}

/// Membership in a submodule is membership in its underlying set.
theorem submodule_contains_as_set_eq[R: Ring, M: AddCommGroup](s: Submodule[R, M], x: M) {
    s.contains(x) = s.as_set.contains(x)
} by {
    submodule_as_set_contains_eq(s, x)
    s.contains(x) = s.as_set.contains(x)
}

/// Equal submodules have equal underlying sets.
theorem submodule_eq_as_set[R: Ring, M: AddCommGroup](
    a: Submodule[R, M], b: Submodule[R, M]
) {
    a = b implies a.as_set = b.as_set
}

/// True if every element of one submodule belongs to another.
define submodule_subset[R: Ring, M: AddCommGroup](a: Submodule[R, M], b: Submodule[R, M]) -> Bool {
    forall(x: M) {
        a.contains(x) implies b.contains(x)
    }
}

/// Submodule inclusion is reflexive.
theorem submodule_subset_refl[R: Ring, M: AddCommGroup](a: Submodule[R, M]) {
    submodule_subset(a, a)
} by {
    forall(x: M) {
        a.contains(x) implies a.contains(x)
    }
}

/// Submodule inclusion is transitive.
theorem submodule_subset_trans[R: Ring, M: AddCommGroup](
    a: Submodule[R, M],
    b: Submodule[R, M],
    c: Submodule[R, M]
) {
    submodule_subset(a, b) and submodule_subset(b, c) implies submodule_subset(a, c)
} by {
    if submodule_subset(a, b) and submodule_subset(b, c) {
        submodule_subset(a, b) = forall(y: M) {
            a.contains(y) implies b.contains(y)
        }
        submodule_subset(b, c) = forall(y: M) {
            b.contains(y) implies c.contains(y)
        }
        forall(x: M) {
            if a.contains(x) {
                b.contains(x)
                c.contains(x)
            }
        }
    }
}

/// Every common lower bound is contained in the common membership predicate.
theorem submodule_subset_intersection_contains_of_subset_left_right[R: Ring, M: AddCommGroup](
    c: Submodule[R, M],
    a: Submodule[R, M],
    b: Submodule[R, M]
) {
    submodule_subset(c, a) and submodule_subset(c, b) implies forall(x: M) {
        c.contains(x) implies submodule_intersection_contains(a, b, x)
    }
} by {
    if submodule_subset(c, a) and submodule_subset(c, b) {
        submodule_subset(c, a) = forall(y: M) {
            c.contains(y) implies a.contains(y)
        }
        submodule_subset(c, b) = forall(y: M) {
            c.contains(y) implies b.contains(y)
        }
        forall(x: M) {
            if c.contains(x) {
                a.contains(x)
                b.contains(x)
                submodule_intersection_contains_eq(a, b, x)
                submodule_intersection_contains(a, b, x)
            }
        }
    }
}

/// Containment in the common membership predicate gives containment in the left submodule.
theorem submodule_subset_left_of_subset_intersection_contains[R: Ring, M: AddCommGroup](
    c: Submodule[R, M],
    a: Submodule[R, M],
    b: Submodule[R, M]
) {
    (forall(x: M) { c.contains(x) implies submodule_intersection_contains(a, b, x) })
        implies submodule_subset(c, a)
} by {
    if forall(x: M) { c.contains(x) implies submodule_intersection_contains(a, b, x) } {
        forall(x: M) {
            if c.contains(x) {
                submodule_intersection_contains(a, b, x)
                submodule_intersection_subset_left(a, b, x)
                a.contains(x)
            }
        }
    }
}

/// Containment in the common membership predicate gives containment in the right submodule.
theorem submodule_subset_right_of_subset_intersection_contains[R: Ring, M: AddCommGroup](
    c: Submodule[R, M],
    a: Submodule[R, M],
    b: Submodule[R, M]
) {
    (forall(x: M) { c.contains(x) implies submodule_intersection_contains(a, b, x) })
        implies submodule_subset(c, b)
} by {
    if forall(x: M) { c.contains(x) implies submodule_intersection_contains(a, b, x) } {
        forall(x: M) {
            if c.contains(x) {
                submodule_intersection_contains(a, b, x)
                submodule_intersection_subset_right(a, b, x)
                b.contains(x)
            }
        }
    }
}

/// Containment in the common membership predicate is equivalent to containment in both submodules.
theorem submodule_subset_intersection_contains_iff[R: Ring, M: AddCommGroup](
    c: Submodule[R, M],
    a: Submodule[R, M],
    b: Submodule[R, M]
) {
    (forall(x: M) { c.contains(x) implies submodule_intersection_contains(a, b, x) }) =
        (submodule_subset(c, a) and submodule_subset(c, b))
} by {
    if forall(x: M) { c.contains(x) implies submodule_intersection_contains(a, b, x) } {
        submodule_subset_left_of_subset_intersection_contains(c, a, b)
        submodule_subset_right_of_subset_intersection_contains(c, a, b)
        submodule_subset(c, a)
        submodule_subset(c, b)
        submodule_subset(c, a) and submodule_subset(c, b)
    }
    if submodule_subset(c, a) and submodule_subset(c, b) {
        submodule_subset_intersection_contains_of_subset_left_right(c, a, b)
        forall(x: M) { c.contains(x) implies submodule_intersection_contains(a, b, x) }
    }
    (forall(x: M) { c.contains(x) implies submodule_intersection_contains(a, b, x) }) =
        (submodule_subset(c, a) and submodule_subset(c, b))
}

/// Membership in a same-carrier intersection means membership in both submodules.
theorem submodule_intersection_contains_eq_of_carrier_eq[R: Ring, M: AddCommGroup](
    a: Submodule[R, M], b: Submodule[R, M], x: M
) {
    a.carrier = b.carrier implies a.intersection(b).contains(x) = (a.contains(x) and b.contains(x))
} by {
    if a.carrier = b.carrier {
        a.intersection(b) = submodule_intersection(a, b)
        submodule_intersection(a, b).contains(x) = submodule_bundled_intersection_contains(a, b, x)
        a.intersection(b).contains(x) = submodule_bundled_intersection_contains(a, b, x)
        submodule_bundled_intersection_contains_eq_of_carrier_eq(a, b, x)
        a.intersection(b).contains(x) = submodule_intersection_contains(a, b, x)
        submodule_intersection_contains_eq(a, b, x)
        a.intersection(b).contains(x) = (a.contains(x) and b.contains(x))
    }
}

/// The same-carrier intersection is contained in the left submodule.
theorem submodule_intersection_subset_left_relation[R: Ring, M: AddCommGroup](
    a: Submodule[R, M], b: Submodule[R, M]
) {
    a.carrier = b.carrier implies submodule_subset(a.intersection(b), a)
} by {
    if a.carrier = b.carrier {
        forall(x: M) {
            if a.intersection(b).contains(x) {
                submodule_intersection_contains_eq_of_carrier_eq(a, b, x)
                a.contains(x)
            }
        }
    }
}

/// The same-carrier intersection is contained in the right submodule.
theorem submodule_intersection_subset_right_relation[R: Ring, M: AddCommGroup](
    a: Submodule[R, M], b: Submodule[R, M]
) {
    a.carrier = b.carrier implies submodule_subset(a.intersection(b), b)
} by {
    if a.carrier = b.carrier {
        forall(x: M) {
            if a.intersection(b).contains(x) {
                submodule_intersection_contains_eq_of_carrier_eq(a, b, x)
                b.contains(x)
            }
        }
    }
}

/// Every common submodule of two same-carrier submodules is contained in their intersection.
theorem submodule_subset_intersection_of_subset_left_right_bundled[R: Ring, M: AddCommGroup](
    c: Submodule[R, M],
    a: Submodule[R, M],
    b: Submodule[R, M]
) {
    c.carrier = a.carrier and a.carrier = b.carrier and submodule_subset(c, a) and submodule_subset(c, b)
    implies submodule_subset(c, a.intersection(b))
} by {
    if c.carrier = a.carrier and a.carrier = b.carrier and submodule_subset(c, a) and submodule_subset(c, b) {
        forall(x: M) {
            if c.contains(x) {
                a.contains(x)
                b.contains(x)
                submodule_intersection_contains_eq_of_carrier_eq(a, b, x)
                a.intersection(b).contains(x)
            }
        }
    }
}

/// Containment in a same-carrier intersection is equivalent to containment in both submodules.
theorem submodule_subset_intersection_iff_bundled[R: Ring, M: AddCommGroup](
    c: Submodule[R, M],
    a: Submodule[R, M],
    b: Submodule[R, M]
) {
    c.carrier = a.carrier and a.carrier = b.carrier implies
        (submodule_subset(c, a.intersection(b)) = (submodule_subset(c, a) and submodule_subset(c, b)))
} by {
    if c.carrier = a.carrier and a.carrier = b.carrier {
        if submodule_subset(c, a.intersection(b)) {
            submodule_intersection_subset_left_relation(a, b)
            submodule_subset_trans(c, a.intersection(b), a)
            submodule_subset(c, a)
            submodule_intersection_subset_right_relation(a, b)
            submodule_subset_trans(c, a.intersection(b), b)
            submodule_subset(c, b)
            submodule_subset(c, a) and submodule_subset(c, b)
        }
        if submodule_subset(c, a) and submodule_subset(c, b) {
            submodule_subset_intersection_of_subset_left_right_bundled(c, a, b)
            submodule_subset(c, a.intersection(b))
        }
        submodule_subset(c, a.intersection(b)) = (submodule_subset(c, a) and submodule_subset(c, b))
    }
}

/// Same-carrier submodule intersection is idempotent.
theorem submodule_intersection_idempotent[R: Ring, M: AddCommGroup](a: Submodule[R, M]) {
    a.intersection(a) = a
} by {
    forall(x: M) {
        if a.intersection(a).contains(x) {
            submodule_intersection_contains_eq_of_carrier_eq(a, a, x)
            a.contains(x)
        }
        if a.contains(x) {
            submodule_intersection_contains_eq_of_carrier_eq(a, a, x)
            a.intersection(a).contains(x)
        }
        a.intersection(a).contains(x) = a.contains(x)
    }
    a.intersection(a) = submodule_intersection(a, a)
    submodule_intersection_carrier_eq(a, a)
    a.intersection(a).carrier = a.carrier
    submodule_ext(a.intersection(a), a)
}

/// Mutual inclusion gives equality of membership predicates.
theorem submodule_contains_eq_of_subset_both[R: Ring, M: AddCommGroup](a: Submodule[R, M], b: Submodule[R, M]) {
    submodule_subset(a, b) and submodule_subset(b, a) implies a.contains = b.contains
} by {
    if submodule_subset(a, b) and submodule_subset(b, a) {
        submodule_subset(a, b) = forall(y: M) {
            a.contains(y) implies b.contains(y)
        }
        submodule_subset(b, a) = forall(y: M) {
            b.contains(y) implies a.contains(y)
        }
        forall(x: M) {
            if a.contains(x) {
                b.contains(x)
            }
            if b.contains(x) {
                a.contains(x)
            }
            a.contains(x) = b.contains(x)
        }
        predicate_extensionality(a.contains, b.contains)
    }
}

/// Mutual inclusion of submodules forces equality.
theorem submodule_subset_antisymm[R: Ring, M: AddCommGroup](a: Submodule[R, M], b: Submodule[R, M]) {
    a.carrier = b.carrier and submodule_subset(a, b) and submodule_subset(b, a) implies a = b
} by {
    if a.carrier = b.carrier and submodule_subset(a, b) and submodule_subset(b, a) {
        submodule_contains_eq_of_subset_both(a, b)
        a.contains = b.contains
        submodule_ext(a, b)
    }
}

/// Equality of submodules is equivalent to mutual inclusion.
theorem submodule_eq_iff_subset_both[R: Ring, M: AddCommGroup](a: Submodule[R, M], b: Submodule[R, M]) {
    a.carrier = b.carrier implies (a = b = (submodule_subset(a, b) and submodule_subset(b, a)))
} by {
    if a.carrier = b.carrier {
        if a = b {
            submodule_subset_refl(a)
            submodule_subset(a, b)
            submodule_subset(b, a)
            submodule_subset(a, b) and submodule_subset(b, a)
        }
        if submodule_subset(a, b) and submodule_subset(b, a) {
            submodule_subset_antisymm(a, b)
            a = b
        }
        a = b = (submodule_subset(a, b) and submodule_subset(b, a))
    }
}

/// The bundled common part is symmetric on equal carriers.
theorem submodule_intersection_contains_comm_pointwise[R: Ring, M: AddCommGroup](
    a: Submodule[R, M], b: Submodule[R, M], x: M
) {
    a.carrier = b.carrier implies a.intersection(b).contains(x) = b.intersection(a).contains(x)
} by {
    if a.carrier = b.carrier {
        submodule_intersection_contains_comm_at(a, b, x)
        submodule_intersection_contains_eq_of_carrier_eq(a, b, x)
        submodule_intersection_contains_eq_of_carrier_eq(b, a, x)
        submodule_intersection_contains_eq(a, b, x)
        submodule_intersection_contains_eq(b, a, x)
        a.intersection(b).contains(x) = submodule_intersection_contains(a, b, x)
        b.intersection(a).contains(x) = submodule_intersection_contains(b, a, x)
        a.intersection(b).contains(x) = b.intersection(a).contains(x)
    }
}

/// The bundled common part membership predicate is symmetric on equal carriers.
theorem submodule_intersection_contains_comm_pred[R: Ring, M: AddCommGroup](
    a: Submodule[R, M], b: Submodule[R, M]
) {
    a.carrier = b.carrier implies a.intersection(b).contains = b.intersection(a).contains
} by {
    if a.carrier = b.carrier {
        forall(x: M) {
            submodule_intersection_contains_comm_pointwise(a, b, x)
        }
        predicate_extensionality(a.intersection(b).contains, b.intersection(a).contains)
    }
}

/// Same-carrier submodule intersection is commutative.
theorem submodule_intersection_comm[R: Ring, M: AddCommGroup](
    a: Submodule[R, M], b: Submodule[R, M]
) {
    a.carrier = b.carrier implies a.intersection(b) = b.intersection(a)
} by {
    if a.carrier = b.carrier {
        submodule_intersection_carrier_eq(a, b)
        submodule_intersection_carrier_eq(b, a)
        submodule_intersection_contains_comm_pred(a, b)
        a.intersection(b).contains = b.intersection(a).contains
        a.intersection(b).carrier = b.intersection(a).carrier
    }
}

/// The bundled triple intersection associates pointwise on equal carriers.
theorem submodule_intersection_contains_assoc_pointwise[R: Ring, M: AddCommGroup](
    a: Submodule[R, M], b: Submodule[R, M], c: Submodule[R, M], x: M
) {
    a.carrier = b.carrier and b.carrier = c.carrier implies
        a.intersection(b).intersection(c).contains(x) = a.intersection(b.intersection(c)).contains(x)
} by {
    if a.carrier = b.carrier and b.carrier = c.carrier {
        submodule_intersection_carrier_eq(a, b)
        submodule_intersection_carrier_eq(b, c)
        a.intersection(b).carrier = c.carrier
        a.carrier = b.intersection(c).carrier
        submodule_intersection_contains_eq_of_carrier_eq(a.intersection(b), c, x)
        submodule_intersection_contains_eq_of_carrier_eq(a, b, x)
        submodule_intersection_contains_eq_of_carrier_eq(b, c, x)
        submodule_intersection_contains_eq_of_carrier_eq(a, b.intersection(c), x)
        if a.intersection(b).intersection(c).contains(x) {
            a.intersection(b).contains(x)
            c.contains(x)
            a.contains(x)
            b.contains(x)
            b.intersection(c).contains(x)
            a.intersection(b.intersection(c)).contains(x)
        }
        if a.intersection(b.intersection(c)).contains(x) {
            a.contains(x)
            b.intersection(c).contains(x)
            b.contains(x)
            c.contains(x)
            a.intersection(b).contains(x)
            a.intersection(b).intersection(c).contains(x)
        }
        a.intersection(b).intersection(c).contains(x) = a.intersection(b.intersection(c)).contains(x)
    }
}

/// The bundled triple intersection associates as a predicate on equal carriers.
theorem submodule_intersection_contains_assoc_pred[R: Ring, M: AddCommGroup](
    a: Submodule[R, M], b: Submodule[R, M], c: Submodule[R, M]
) {
    a.carrier = b.carrier and b.carrier = c.carrier implies
        a.intersection(b).intersection(c).contains = a.intersection(b.intersection(c)).contains
} by {
    if a.carrier = b.carrier and b.carrier = c.carrier {
        forall(x: M) {
            submodule_intersection_contains_assoc_pointwise(a, b, c, x)
        }
        predicate_extensionality(
            a.intersection(b).intersection(c).contains,
            a.intersection(b.intersection(c)).contains
        )
    }
}

/// The carrier of an iterated bundled intersection is the leftmost carrier.
theorem submodule_intersection_assoc_carrier_eq[R: Ring, M: AddCommGroup](
    a: Submodule[R, M], b: Submodule[R, M], c: Submodule[R, M]
) {
    a.intersection(b).intersection(c).carrier = a.intersection(b.intersection(c)).carrier
} by {
    submodule_intersection_carrier_eq(a, b)
    submodule_intersection_carrier_eq(a.intersection(b), c)
    submodule_intersection_carrier_eq(a, b.intersection(c))
    a.intersection(b).intersection(c).carrier = a.carrier
    a.intersection(b.intersection(c)).carrier = a.carrier
}

/// Same-carrier submodule intersection is associative.
theorem submodule_intersection_assoc[R: Ring, M: AddCommGroup](
    a: Submodule[R, M], b: Submodule[R, M], c: Submodule[R, M]
) {
    a.carrier = b.carrier and b.carrier = c.carrier implies
        a.intersection(b).intersection(c) = a.intersection(b.intersection(c))
} by {
    if a.carrier = b.carrier and b.carrier = c.carrier {
        submodule_intersection_assoc_carrier_eq(a, b, c)
        submodule_intersection_contains_assoc_pred(a, b, c)
        submodule_eq_of_carrier_contains_eq(
            a.intersection(b).intersection(c),
            a.intersection(b.intersection(c))
        )
    }
}

/// Intersecting with a larger same-carrier submodule on the right gives the smaller submodule.
theorem submodule_intersection_eq_left_of_subset[R: Ring, M: AddCommGroup](
    a: Submodule[R, M], b: Submodule[R, M]
) {
    a.carrier = b.carrier and submodule_subset(a, b) implies a.intersection(b) = a
} by {
    if a.carrier = b.carrier and submodule_subset(a, b) {
        submodule_intersection_carrier_eq(a, b)
        a.intersection(b).carrier = a.carrier
        forall(x: M) {
            if a.intersection(b).contains(x) {
                submodule_intersection_contains_eq_of_carrier_eq(a, b, x)
                a.contains(x)
            }
            if a.contains(x) {
                submodule_subset(a, b) = forall(y: M) {
                    a.contains(y) implies b.contains(y)
                }
                b.contains(x)
                submodule_intersection_contains_eq_of_carrier_eq(a, b, x)
                a.intersection(b).contains(x)
            }
            a.intersection(b).contains(x) = a.contains(x)
        }
        predicate_extensionality(a.intersection(b).contains, a.contains)
        a.intersection(b).contains = a.contains
    }
}

/// Intersecting with a larger same-carrier submodule on the left gives the smaller submodule.
theorem submodule_intersection_eq_right_of_subset[R: Ring, M: AddCommGroup](
    a: Submodule[R, M], b: Submodule[R, M]
) {
    a.carrier = b.carrier and submodule_subset(b, a) implies a.intersection(b) = b
} by {
    if a.carrier = b.carrier and submodule_subset(b, a) {
        submodule_intersection_comm(a, b)
        a.intersection(b) = b.intersection(a)
        b.carrier = a.carrier
        submodule_intersection_eq_left_of_subset(b, a)
        b.intersection(a) = b
        a.intersection(b) = b
    }
}

/// True if an element is the additive identity.
define zero_submodule_contains[M: AddCommGroup](x: M) -> Bool {
    x = M.0
}

/// The additive identity predicate is a submodule predicate.
theorem zero_submodule_is_submodule[R: Ring, M: AddCommGroup](carrier: Module[R, M]) {
    is_submodule(carrier, zero_submodule_contains[M])
} by {
    zero_submodule_contains[M](M.0)
    submodule_zero_constraint(carrier, zero_submodule_contains[M])
    forall(x: M, y: M) {
        if zero_submodule_contains[M](x) and zero_submodule_contains[M](y) {
            x = M.0
            y = M.0
            M.0 + M.0 = M.0
            zero_submodule_contains[M](x + y)
        }
    }
    submodule_add_constraint(carrier, zero_submodule_contains[M])
    forall(x: M) {
        if zero_submodule_contains[M](x) {
            x = M.0
            -M.0 = M.0
            zero_submodule_contains[M](-x)
        }
    }
    submodule_neg_constraint(carrier, zero_submodule_contains[M])
    forall(r: R, x: M) {
        if zero_submodule_contains[M](x) {
            x = M.0
            module_smul_zero_right(carrier, r)
            carrier.smul(r, M.0) = M.0
            zero_submodule_contains[M](carrier.smul(r, x))
        }
    }
    submodule_smul_constraint(carrier, zero_submodule_contains[M])
}

/// The zero submodule of a module.
let zero_submodule[R: Ring, M: AddCommGroup](carrier: Module[R, M]) -> result: Submodule[R, M] satisfy {
    Submodule.new(carrier, zero_submodule_contains[M]) = Option.some(result)
} by {
    zero_submodule_is_submodule(carrier)
}

/// Membership in the zero submodule is equality to the additive identity.
theorem zero_submodule_contains_eq[R: Ring, M: AddCommGroup](carrier: Module[R, M], x: M) {
    zero_submodule(carrier).contains(x) = (x = M.0)
} by {
    zero_submodule(carrier).contains(x) = zero_submodule_contains[M](x)
    zero_submodule_contains[M](x) = (x = M.0)
}

/// The zero submodule has the prescribed ambient module.
theorem zero_submodule_carrier_eq[R: Ring, M: AddCommGroup](carrier: Module[R, M]) {
    zero_submodule(carrier).carrier = carrier
}

/// The zero submodule is contained in every submodule of its module.
theorem zero_submodule_subset[R: Ring, M: AddCommGroup](s: Submodule[R, M]) {
    submodule_subset(zero_submodule(s.carrier), s)
} by {
    forall(x: M) {
        if zero_submodule(s.carrier).contains(x) {
            zero_submodule_contains_eq(s.carrier, x)
            x = M.0
            submodule_contains_zero(s)
            s.contains(M.0)
            s.contains(x)
        }
    }
}

/// The universal predicate on a module.
define full_submodule_contains[M: AddCommGroup](x: M) -> Bool {
    true
}

/// The universal predicate is a submodule predicate.
theorem full_submodule_is_submodule[R: Ring, M: AddCommGroup](carrier: Module[R, M]) {
    is_submodule(carrier, full_submodule_contains[M])
} by {
    full_submodule_contains[M](M.0)
    submodule_zero_constraint(carrier, full_submodule_contains[M])
    forall(x: M, y: M) {
        if full_submodule_contains[M](x) and full_submodule_contains[M](y) {
            full_submodule_contains[M](x + y)
        }
    }
    submodule_add_constraint(carrier, full_submodule_contains[M])
    forall(x: M) {
        if full_submodule_contains[M](x) {
            full_submodule_contains[M](-x)
        }
    }
    submodule_neg_constraint(carrier, full_submodule_contains[M])
    forall(r: R, x: M) {
        if full_submodule_contains[M](x) {
            full_submodule_contains[M](carrier.smul(r, x))
        }
    }
    submodule_smul_constraint(carrier, full_submodule_contains[M])
}

/// The full submodule of a module.
let full_submodule[R: Ring, M: AddCommGroup](carrier: Module[R, M]) -> result: Submodule[R, M] satisfy {
    Submodule.new(carrier, full_submodule_contains[M]) = Option.some(result)
} by {
    full_submodule_is_submodule(carrier)
}

/// Every element belongs to the full submodule.
theorem full_submodule_contains_everything[R: Ring, M: AddCommGroup](carrier: Module[R, M], x: M) {
    full_submodule(carrier).contains(x)
}

/// Membership in the full submodule is always true.
theorem full_submodule_contains_eq[R: Ring, M: AddCommGroup](carrier: Module[R, M], x: M) {
    full_submodule(carrier).contains(x) = true
} by {
    full_submodule_contains_everything(carrier, x)
}

/// The full submodule has the prescribed ambient module.
theorem full_submodule_carrier_eq[R: Ring, M: AddCommGroup](carrier: Module[R, M]) {
    full_submodule(carrier).carrier = carrier
}

/// Every submodule is contained in the full submodule of its module.
theorem submodule_subset_full[R: Ring, M: AddCommGroup](s: Submodule[R, M]) {
    submodule_subset(s, full_submodule(s.carrier))
} by {
    forall(x: M) {
        if s.contains(x) {
            full_submodule_contains_everything(s.carrier, x)
        }
    }
}

/// Common membership with the zero submodule on the left is zero-submodule membership.
theorem zero_submodule_intersection_contains_left[R: Ring, M: AddCommGroup](
    carrier: Module[R, M], s: Submodule[R, M], x: M
) {
    s.carrier = carrier implies
        submodule_intersection_contains(zero_submodule(carrier), s, x) = zero_submodule(carrier).contains(x)
} by {
    if s.carrier = carrier {
        if submodule_intersection_contains(zero_submodule(carrier), s, x) {
            submodule_intersection_contains_eq(zero_submodule(carrier), s, x)
            zero_submodule(carrier).contains(x)
        }
        if zero_submodule(carrier).contains(x) {
            zero_submodule_contains_eq(carrier, x)
            x = M.0
            submodule_contains_zero(s)
            s.contains(M.0)
            s.contains(x)
            submodule_intersection_contains_eq(zero_submodule(carrier), s, x)
            submodule_intersection_contains(zero_submodule(carrier), s, x)
        }
        submodule_intersection_contains(zero_submodule(carrier), s, x) = zero_submodule(carrier).contains(x)
    }
}

/// Common membership with the zero submodule on the right is zero-submodule membership.
theorem zero_submodule_intersection_contains_right[R: Ring, M: AddCommGroup](
    carrier: Module[R, M], s: Submodule[R, M], x: M
) {
    s.carrier = carrier implies
        submodule_intersection_contains(s, zero_submodule(carrier), x) = zero_submodule(carrier).contains(x)
} by {
    if s.carrier = carrier {
        submodule_intersection_contains_comm_at(s, zero_submodule(carrier), x)
        zero_submodule_intersection_contains_left(carrier, s, x)
        submodule_intersection_contains(s, zero_submodule(carrier), x) = zero_submodule(carrier).contains(x)
    }
}

/// Common membership with the full submodule on the left is ordinary membership.
theorem full_submodule_intersection_contains_left[R: Ring, M: AddCommGroup](
    carrier: Module[R, M], s: Submodule[R, M], x: M
) {
    submodule_intersection_contains(full_submodule(carrier), s, x) = s.contains(x)
} by {
    full_submodule_contains_everything(carrier, x)
    submodule_intersection_contains_eq(full_submodule(carrier), s, x)
    submodule_intersection_contains(full_submodule(carrier), s, x) = s.contains(x)
}

/// Common membership with the full submodule on the right is ordinary membership.
theorem full_submodule_intersection_contains_right[R: Ring, M: AddCommGroup](
    carrier: Module[R, M], s: Submodule[R, M], x: M
) {
    submodule_intersection_contains(s, full_submodule(carrier), x) = s.contains(x)
} by {
    submodule_intersection_contains_comm_at(s, full_submodule(carrier), x)
    full_submodule_intersection_contains_left(carrier, s, x)
    submodule_intersection_contains(s, full_submodule(carrier), x) = s.contains(x)
}

/// The zero submodule is absorbing under intersection on the left.
theorem zero_submodule_intersection_left[R: Ring, M: AddCommGroup](s: Submodule[R, M]) {
    zero_submodule(s.carrier).intersection(s) = zero_submodule(s.carrier)
} by {
    zero_submodule_carrier_eq(s.carrier)
    zero_submodule_subset(s)
    submodule_intersection_eq_left_of_subset(zero_submodule(s.carrier), s)
}

/// The zero submodule is absorbing under intersection on the right.
theorem zero_submodule_intersection_right[R: Ring, M: AddCommGroup](s: Submodule[R, M]) {
    s.intersection(zero_submodule(s.carrier)) = zero_submodule(s.carrier)
} by {
    zero_submodule_carrier_eq(s.carrier)
    zero_submodule_subset(s)
    submodule_intersection_eq_right_of_subset(s, zero_submodule(s.carrier))
}

/// The full submodule is the identity for intersection on the left.
theorem full_submodule_intersection_left[R: Ring, M: AddCommGroup](s: Submodule[R, M]) {
    full_submodule(s.carrier).intersection(s) = s
} by {
    full_submodule_carrier_eq(s.carrier)
    submodule_subset_full(s)
    submodule_intersection_eq_right_of_subset(full_submodule(s.carrier), s)
}

/// The full submodule is the identity for intersection on the right.
theorem full_submodule_intersection_right[R: Ring, M: AddCommGroup](s: Submodule[R, M]) {
    s.intersection(full_submodule(s.carrier)) = s
} by {
    full_submodule_carrier_eq(s.carrier)
    submodule_subset_full(s)
    submodule_intersection_eq_left_of_subset(s, full_submodule(s.carrier))
}

/// True if every element of a set belongs to a submodule.
define set_subset_submodule[R: Ring, M: AddCommGroup](a: Set[M], s: Submodule[R, M]) -> Bool {
    forall(x: M) {
        a.contains(x) implies s.contains(x)
    }
}

/// Set containment in a submodule is containment in its underlying set.
theorem set_subset_submodule_as_set_eq[R: Ring, M: AddCommGroup](a: Set[M], s: Submodule[R, M]) {
    set_subset_submodule(a, s) = a.subset(s.as_set)
} by {
    if set_subset_submodule(a, s) {
        forall(x: M) {
            if a.contains(x) {
                s.contains(x)
                submodule_as_set_contains_eq(s, x)
                s.as_set.contains(x)
            }
        }
        a.subset(s.as_set)
    }
    if a.subset(s.as_set) {
        forall(x: M) {
            if a.contains(x) {
                a.subset(s.as_set) = forall(y: M) {
                    a.contains(y) implies s.as_set.contains(y)
                }
                s.as_set.contains(x)
                submodule_as_set_contains_eq(s, x)
                s.contains(x)
            }
        }
        set_subset_submodule(a, s)
    }
    set_subset_submodule(a, s) = a.subset(s.as_set)
}

/// Set containment in a submodule gives containment in its underlying set.
theorem set_as_set_subset_of_subset_submodule[R: Ring, M: AddCommGroup](a: Set[M], s: Submodule[R, M]) {
    set_subset_submodule(a, s) implies a.subset(s.as_set)
} by {
    if set_subset_submodule(a, s) {
        set_subset_submodule_as_set_eq(a, s)
        a.subset(s.as_set)
    }
}

/// Containment in the underlying set gives set containment in the submodule.
theorem set_subset_submodule_of_as_set_subset[R: Ring, M: AddCommGroup](a: Set[M], s: Submodule[R, M]) {
    a.subset(s.as_set) implies set_subset_submodule(a, s)
} by {
    if a.subset(s.as_set) {
        set_subset_submodule_as_set_eq(a, s)
        set_subset_submodule(a, s)
    }
}

/// Submodule containment is containment of the underlying sets.
theorem submodule_subset_as_set_eq[R: Ring, M: AddCommGroup](a: Submodule[R, M], b: Submodule[R, M]) {
    submodule_subset(a, b) = a.as_set.subset(b.as_set)
} by {
    if submodule_subset(a, b) {
        forall(x: M) {
            if a.as_set.contains(x) {
                submodule_as_set_contains_eq(a, x)
                a.contains(x)
                b.contains(x)
                submodule_as_set_contains_eq(b, x)
                b.as_set.contains(x)
            }
        }
        a.as_set.subset(b.as_set)
    }
    if a.as_set.subset(b.as_set) {
        forall(x: M) {
            if a.contains(x) {
                submodule_as_set_contains_eq(a, x)
                a.as_set.contains(x)
                a.as_set.subset(b.as_set) = forall(y: M) {
                    a.as_set.contains(y) implies b.as_set.contains(y)
                }
                b.as_set.contains(x)
                submodule_as_set_contains_eq(b, x)
                b.contains(x)
            }
        }
        submodule_subset(a, b)
    }
    submodule_subset(a, b) = a.as_set.subset(b.as_set)
}

/// Submodule containment gives containment of underlying sets.
theorem submodule_as_set_subset_of_subset[R: Ring, M: AddCommGroup](a: Submodule[R, M], b: Submodule[R, M]) {
    submodule_subset(a, b) implies a.as_set.subset(b.as_set)
} by {
    if submodule_subset(a, b) {
        submodule_subset_as_set_eq(a, b)
        a.as_set.subset(b.as_set)
    }
}

/// Containment of underlying sets gives submodule containment.
theorem submodule_subset_of_as_set_subset[R: Ring, M: AddCommGroup](a: Submodule[R, M], b: Submodule[R, M]) {
    a.as_set.subset(b.as_set) implies submodule_subset(a, b)
} by {
    if a.as_set.subset(b.as_set) {
        submodule_subset_as_set_eq(a, b)
        submodule_subset(a, b)
    }
}

/// The underlying set of a submodule is contained in the submodule.
theorem submodule_as_set_subset_self[R: Ring, M: AddCommGroup](s: Submodule[R, M]) {
    set_subset_submodule(s.as_set, s)
} by {
    forall(x: M) {
        if s.as_set.contains(x) {
            s.contains(x)
        }
    }
}

/// Set containment is preserved by enlarging the target submodule.
theorem set_subset_submodule_trans[R: Ring, M: AddCommGroup](
    a: Set[M],
    s: Submodule[R, M],
    t: Submodule[R, M]
) {
    set_subset_submodule(a, s) and submodule_subset(s, t) implies set_subset_submodule(a, t)
} by {
    if set_subset_submodule(a, s) and submodule_subset(s, t) {
        forall(x: M) {
            if a.contains(x) {
                s.contains(x)
                submodule_subset(s, t) = predicate_subset(s.contains, t.contains)
                predicate_subset(s.contains, t.contains)
                predicate_subset_step(s.contains, t.contains, x)
                t.contains(x)
            }
        }
    }
}

/// Set containment in a submodule is preserved by enlarging the set.
theorem set_subset_submodule_of_set_subset[R: Ring, M: AddCommGroup](
    a: Set[M],
    b: Set[M],
    s: Submodule[R, M]
) {
    a.subset(b) and set_subset_submodule(b, s) implies set_subset_submodule(a, s)
} by {
    if a.subset(b) and set_subset_submodule(b, s) {
        forall(x: M) {
            if a.contains(x) {
                a.subset(b) = forall(y: M) {
                    a.contains(y) implies b.contains(y)
                }
                b.contains(x)
                s.contains(x)
            }
        }
    }
}

/// True if an element belongs to every submodule that contains a given set.
define submodule_closure_contains[R: Ring, M: AddCommGroup](
    carrier: Module[R, M], a: Set[M], x: M
) -> Bool {
    forall(s: Submodule[R, M]) {
        s.carrier = carrier and set_subset_submodule(a, s) implies s.contains(x)
    }
}

/// Membership in the closure gives membership in each submodule that contains the set.
theorem submodule_closure_contains_of_set_subset_raw[R: Ring, M: AddCommGroup](
    carrier: Module[R, M],
    a: Set[M],
    s: Submodule[R, M],
    x: M
) {
    submodule_closure_contains(carrier, a, x) and s.carrier = carrier and set_subset_submodule(a, s)
    implies s.contains(x)
} by {
    if submodule_closure_contains(carrier, a, x) and s.carrier = carrier and set_subset_submodule(a, s) {
        submodule_closure_contains(carrier, a, x) = forall(t: Submodule[R, M]) {
            t.carrier = carrier and set_subset_submodule(a, t) implies t.contains(x)
        }
        s.contains(x)
    }
}

/// The submodule closure of a set contains the additive identity.
theorem submodule_closure_zero_constraint[R: Ring, M: AddCommGroup](carrier: Module[R, M], a: Set[M]) {
    submodule_zero_constraint(carrier, submodule_closure_contains(carrier, a))
} by {
    forall(s: Submodule[R, M]) {
        if s.carrier = carrier and set_subset_submodule(a, s) {
            submodule_contains_zero(s)
            s.contains(M.0)
        }
    }
    submodule_closure_contains(carrier, a, M.0)
    submodule_zero_constraint(carrier, submodule_closure_contains(carrier, a))
}

/// The submodule closure of a set is closed under addition.
theorem submodule_closure_add_constraint[R: Ring, M: AddCommGroup](carrier: Module[R, M], a: Set[M]) {
    submodule_add_constraint(carrier, submodule_closure_contains(carrier, a))
} by {
    forall(x: M, y: M) {
        if submodule_closure_contains(carrier, a, x) and submodule_closure_contains(carrier, a, y) {
            forall(s: Submodule[R, M]) {
                if s.carrier = carrier and set_subset_submodule(a, s) {
                    submodule_closure_contains_of_set_subset_raw(carrier, a, s, x)
                    s.contains(x)
                    submodule_closure_contains_of_set_subset_raw(carrier, a, s, y)
                    s.contains(y)
                    submodule_contains_add(s, x, y)
                    s.contains(x + y)
                }
            }
            submodule_closure_contains(carrier, a, x + y)
        }
    }
}

/// The submodule closure of a set is closed under additive negation.
theorem submodule_closure_neg_constraint[R: Ring, M: AddCommGroup](carrier: Module[R, M], a: Set[M]) {
    submodule_neg_constraint(carrier, submodule_closure_contains(carrier, a))
} by {
    forall(x: M) {
        if submodule_closure_contains(carrier, a, x) {
            forall(s: Submodule[R, M]) {
                if s.carrier = carrier and set_subset_submodule(a, s) {
                    submodule_closure_contains_of_set_subset_raw(carrier, a, s, x)
                    s.contains(x)
                    submodule_contains_neg(s, x)
                    s.contains(-x)
                }
            }
            submodule_closure_contains(carrier, a, -x)
        }
    }
}

/// The submodule closure of a set is closed under scalar multiplication.
theorem submodule_closure_smul_constraint[R: Ring, M: AddCommGroup](carrier: Module[R, M], a: Set[M]) {
    submodule_smul_constraint(carrier, submodule_closure_contains(carrier, a))
} by {
    forall(r: R, x: M) {
        if submodule_closure_contains(carrier, a, x) {
            forall(s: Submodule[R, M]) {
                if s.carrier = carrier and set_subset_submodule(a, s) {
                    submodule_closure_contains_of_set_subset_raw(carrier, a, s, x)
                    s.contains(x)
                    submodule_contains_smul(s, r, x)
                    s.carrier.smul(r, x) = carrier.smul(r, x)
                    s.contains(carrier.smul(r, x))
                }
            }
            submodule_closure_contains(carrier, a, carrier.smul(r, x))
        }
    }
}

/// The submodule closure of a set is a submodule.
theorem submodule_closure_is_submodule[R: Ring, M: AddCommGroup](carrier: Module[R, M], a: Set[M]) {
    is_submodule(carrier, submodule_closure_contains(carrier, a))
} by {
    submodule_closure_zero_constraint(carrier, a)
    submodule_closure_add_constraint(carrier, a)
    submodule_closure_neg_constraint(carrier, a)
    submodule_closure_smul_constraint(carrier, a)
}

/// The smallest submodule of a module containing a set.
let submodule_closure[R: Ring, M: AddCommGroup](carrier: Module[R, M], a: Set[M]) -> result: Submodule[R, M] satisfy {
    Submodule.new(carrier, submodule_closure_contains(carrier, a)) = Option.some(result)
} by {
    submodule_closure_is_submodule(carrier, a)
}

/// Membership in the closure means membership in every submodule containing the set.
theorem submodule_closure_contains_eq[R: Ring, M: AddCommGroup](
    carrier: Module[R, M], a: Set[M], x: M
) {
    submodule_closure(carrier, a).contains(x) = submodule_closure_contains(carrier, a, x)
} by {
    submodule_closure(carrier, a).contains(x) = submodule_closure_contains(carrier, a, x)
}

/// Every generator belongs to the submodule closure.
theorem submodule_subset_closure[R: Ring, M: AddCommGroup](carrier: Module[R, M], a: Set[M]) {
    set_subset_submodule(a, submodule_closure(carrier, a))
} by {
    forall(x: M) {
        if a.contains(x) {
            forall(s: Submodule[R, M]) {
                if s.carrier = carrier and set_subset_submodule(a, s) {
                    s.contains(x)
                }
            }
            submodule_closure_contains(carrier, a, x)
            submodule_closure_contains_eq(carrier, a, x)
            submodule_closure(carrier, a).contains(x)
        }
    }
}

/// A generator is a member of the submodule closure.
theorem submodule_closure_contains_of_set_contains[R: Ring, M: AddCommGroup](
    carrier: Module[R, M], a: Set[M], x: M
) {
    a.contains(x) implies submodule_closure(carrier, a).contains(x)
} by {
    if a.contains(x) {
        submodule_subset_closure(carrier, a)
        submodule_closure(carrier, a).contains(x)
    }
}

/// The submodule closure is contained in any submodule containing the set.
theorem submodule_closure_subset_of_set_subset[R: Ring, M: AddCommGroup](
    carrier: Module[R, M],
    a: Set[M],
    s: Submodule[R, M]
) {
    s.carrier = carrier and set_subset_submodule(a, s) implies submodule_subset(submodule_closure(carrier, a), s)
} by {
    if s.carrier = carrier and set_subset_submodule(a, s) {
        forall(x: M) {
            if submodule_closure(carrier, a).contains(x) {
                submodule_closure_contains_eq(carrier, a, x)
                submodule_closure_contains(carrier, a, x)
                submodule_closure_contains_of_set_subset_raw(carrier, a, s, x)
                s.contains(x)
            }
        }
        submodule_subset(submodule_closure(carrier, a), s)
    }
}

/// The submodule closure is the least submodule containing the set.
theorem submodule_closure_le_iff_set_subset[R: Ring, M: AddCommGroup](
    carrier: Module[R, M],
    a: Set[M],
    s: Submodule[R, M]
) {
    s.carrier = carrier implies
        (submodule_subset(submodule_closure(carrier, a), s) = set_subset_submodule(a, s))
} by {
    if s.carrier = carrier {
        if submodule_subset(submodule_closure(carrier, a), s) {
            submodule_subset_closure(carrier, a)
            set_subset_submodule_trans(a, submodule_closure(carrier, a), s)
            set_subset_submodule(a, s)
        }
        if set_subset_submodule(a, s) {
            submodule_closure_subset_of_set_subset(carrier, a, s)
            submodule_subset(submodule_closure(carrier, a), s)
        }
        submodule_subset(submodule_closure(carrier, a), s) = set_subset_submodule(a, s)
    }
}

/// The closure of the underlying set of a submodule is the submodule itself.
theorem submodule_closure_as_set[R: Ring, M: AddCommGroup](s: Submodule[R, M]) {
    submodule_closure(s.carrier, s.as_set) = s
} by {
    submodule_as_set_subset_self(s)
    submodule_closure_subset_of_set_subset(s.carrier, s.as_set, s)
    submodule_subset(submodule_closure(s.carrier, s.as_set), s)
    submodule_subset_closure(s.carrier, s.as_set)
    forall(x: M) {
        if s.contains(x) {
            s.as_set.contains(x)
            submodule_closure(s.carrier, s.as_set).contains(x)
        }
    }
    submodule_subset(s, submodule_closure(s.carrier, s.as_set))
    submodule_subset(s, submodule_closure(s.carrier, s.as_set))
    submodule_subset_antisymm(submodule_closure(s.carrier, s.as_set), s)
}

/// The closure of a set has the prescribed ambient module.
theorem submodule_closure_carrier_eq[R: Ring, M: AddCommGroup](
    carrier: Module[R, M], a: Set[M]
) {
    submodule_closure(carrier, a).carrier = carrier
}

/// Submodule closure is monotone with respect to set inclusion.
theorem submodule_closure_mono[R: Ring, M: AddCommGroup](
    carrier: Module[R, M], a: Set[M], b: Set[M]
) {
    a.subset(b) implies submodule_subset(submodule_closure(carrier, a), submodule_closure(carrier, b))
} by {
    if a.subset(b) {
        submodule_subset_closure(carrier, b)
        set_subset_submodule_of_set_subset(a, b, submodule_closure(carrier, b))
        set_subset_submodule(a, submodule_closure(carrier, b))
        submodule_closure_subset_of_set_subset(carrier, a, submodule_closure(carrier, b))
        submodule_subset(submodule_closure(carrier, a), submodule_closure(carrier, b))
    }
}

/// Equal sets have equal submodule closures.
theorem submodule_closure_eq_of_set_eq[R: Ring, M: AddCommGroup](
    carrier: Module[R, M], a: Set[M], b: Set[M]
) {
    a = b implies submodule_closure(carrier, a) = submodule_closure(carrier, b)
} by {
    if a = b {
        submodule_closure(carrier, a) = submodule_closure(carrier, b)
    }
}

/// Applying submodule closure twice gives the same submodule.
theorem submodule_closure_idempotent[R: Ring, M: AddCommGroup](carrier: Module[R, M], a: Set[M]) {
    submodule_closure(carrier, submodule_closure(carrier, a).as_set) = submodule_closure(carrier, a)
} by {
    submodule_closure_as_set(submodule_closure(carrier, a))
}

/// The set closure induced by submodule generation in a fixed module.
define submodule_set_closure[R: Ring, M: AddCommGroup](carrier: Module[R, M], a: Set[M]) -> Set[M] {
    submodule_closure(carrier, a).as_set
}

/// The submodule set closure is the underlying set of the generated submodule.
theorem submodule_set_closure_at[R: Ring, M: AddCommGroup](carrier: Module[R, M], a: Set[M]) {
    submodule_set_closure(carrier, a) = submodule_closure(carrier, a).as_set
}

/// The submodule set closure contains the original set.
theorem submodule_set_closure_extensive[R: Ring, M: AddCommGroup](carrier: Module[R, M], a: Set[M]) {
    a.subset(submodule_set_closure(carrier, a))
} by {
    submodule_set_closure_at(carrier, a)
    submodule_subset_closure(carrier, a)
    set_subset_submodule_as_set_eq(a, submodule_closure(carrier, a))
    a.subset(submodule_closure(carrier, a).as_set)
    a.subset(submodule_set_closure(carrier, a))
}

/// The submodule set closure preserves inclusion.
theorem submodule_set_closure_mono[R: Ring, M: AddCommGroup](
    carrier: Module[R, M], a: Set[M], b: Set[M]
) {
    a.subset(b) implies submodule_set_closure(carrier, a).subset(submodule_set_closure(carrier, b))
} by {
    if a.subset(b) {
        submodule_closure_mono(carrier, a, b)
        submodule_subset(submodule_closure(carrier, a), submodule_closure(carrier, b))
        submodule_subset_as_set_eq(submodule_closure(carrier, a), submodule_closure(carrier, b))
        submodule_closure(carrier, a).as_set.subset(submodule_closure(carrier, b).as_set)
        submodule_set_closure_at(carrier, a)
        submodule_set_closure_at(carrier, b)
        submodule_set_closure(carrier, a).subset(submodule_set_closure(carrier, b))
    }
}

/// Applying submodule set closure twice gives the same set.
theorem submodule_set_closure_idempotent[R: Ring, M: AddCommGroup](carrier: Module[R, M], a: Set[M]) {
    submodule_set_closure(carrier, submodule_set_closure(carrier, a)) =
        submodule_set_closure(carrier, a)
} by {
    submodule_set_closure_at(carrier, a)
    submodule_set_closure_at(carrier, submodule_set_closure(carrier, a))
    submodule_closure_idempotent(carrier, a)
    submodule_set_closure(carrier, submodule_set_closure(carrier, a)) =
        submodule_set_closure(carrier, a)
}

/// Submodule set closure is monotone as a set map.
theorem submodule_set_closure_is_monotone[R: Ring, M: AddCommGroup](carrier: Module[R, M]) {
    is_subset_monotone_map(submodule_set_closure(carrier))
} by {
    forall(a: Set[M], b: Set[M]) {
        if a.subset(b) {
            submodule_set_closure_mono(carrier, a, b)
            submodule_set_closure(carrier, a).subset(submodule_set_closure(carrier, b))
        }
    }
}

/// Submodule set closure is extensive as a set map.
theorem submodule_set_closure_is_extensive[R: Ring, M: AddCommGroup](carrier: Module[R, M]) {
    is_set_extensive_map(submodule_set_closure(carrier))
} by {
    forall(a: Set[M]) {
        submodule_set_closure_extensive(carrier, a)
        a.subset(submodule_set_closure(carrier, a))
    }
}

/// Submodule set closure is idempotent as a set map.
theorem submodule_set_closure_is_idempotent[R: Ring, M: AddCommGroup](carrier: Module[R, M]) {
    is_set_idempotent_map(submodule_set_closure(carrier))
} by {
    forall(a: Set[M]) {
        submodule_set_closure_idempotent(carrier, a)
        submodule_set_closure(carrier, submodule_set_closure(carrier, a)) =
            submodule_set_closure(carrier, a)
    }
}

/// Submodule generation induces a closure operator on sets.
theorem submodule_set_closure_operator[R: Ring, M: AddCommGroup](carrier: Module[R, M]) {
    is_set_closure_operator(submodule_set_closure(carrier))
} by {
    submodule_set_closure_is_monotone(carrier)
    submodule_set_closure_is_extensive(carrier)
    submodule_set_closure_is_idempotent(carrier)
    is_set_closure_operator(submodule_set_closure(carrier))
}

/// The closure of the empty set is the zero submodule.
theorem submodule_closure_empty[R: Ring, M: AddCommGroup](carrier: Module[R, M]) {
    submodule_closure(carrier, Set[M].empty_set) = zero_submodule(carrier)
} by {
    set_subset_submodule(Set[M].empty_set, zero_submodule(carrier))
    submodule_closure_subset_of_set_subset(carrier, Set[M].empty_set, zero_submodule(carrier))
    submodule_subset(submodule_closure(carrier, Set[M].empty_set), zero_submodule(carrier))
    forall(x: M) {
        if zero_submodule(carrier).contains(x) {
            zero_submodule_contains_eq(carrier, x)
            x = M.0
            submodule_contains_zero(submodule_closure(carrier, Set[M].empty_set))
            submodule_closure(carrier, Set[M].empty_set).contains(M.0)
            submodule_closure(carrier, Set[M].empty_set).contains(x)
        }
    }
    submodule_subset(zero_submodule(carrier), submodule_closure(carrier, Set[M].empty_set))
    submodule_closure_carrier_eq(carrier, Set[M].empty_set)
    zero_submodule_carrier_eq(carrier)
    submodule_subset_antisymm(submodule_closure(carrier, Set[M].empty_set), zero_submodule(carrier))
}

/// The closure of the universal set is contained in the full submodule.
theorem submodule_closure_universal_subset_full[R: Ring, M: AddCommGroup](carrier: Module[R, M]) {
    submodule_subset(submodule_closure(carrier, Set[M].universal_set), full_submodule(carrier))
} by {
    submodule_subset_full(submodule_closure(carrier, Set[M].universal_set))
}

/// The full submodule is contained in the closure of the universal set.
theorem full_submodule_subset_closure_universal[R: Ring, M: AddCommGroup](carrier: Module[R, M]) {
    submodule_subset(full_submodule(carrier), submodule_closure(carrier, Set[M].universal_set))
} by {
    forall(x: M) {
        if full_submodule(carrier).contains(x) {
            Set[M].universal_set.contains(x)
            submodule_closure_contains_of_set_contains(carrier, Set[M].universal_set, x)
            submodule_closure(carrier, Set[M].universal_set).contains(x)
        }
    }
}

/// The closure of the universal set is the full submodule.
theorem submodule_closure_universal[R: Ring, M: AddCommGroup](carrier: Module[R, M]) {
    submodule_closure(carrier, Set[M].universal_set) = full_submodule(carrier)
} by {
    submodule_closure_universal_subset_full(carrier)
    full_submodule_subset_closure_universal(carrier)
    submodule_closure_carrier_eq(carrier, Set[M].universal_set)
    full_submodule_carrier_eq(carrier)
    submodule_subset_antisymm(submodule_closure(carrier, Set[M].universal_set), full_submodule(carrier))
}

/// The least submodule containing two submodules of the same ambient module.
define submodule_sup[R: Ring, M: AddCommGroup](a: Submodule[R, M], b: Submodule[R, M]) -> Submodule[R, M] {
    submodule_closure(a.carrier, a.as_set.union(b.as_set))
}

/// The join of two submodules is the closure of the union of their underlying sets.
theorem submodule_sup_eq_closure_union[R: Ring, M: AddCommGroup](
    a: Submodule[R, M], b: Submodule[R, M]
) {
    submodule_sup(a, b) = submodule_closure(a.carrier, a.as_set.union(b.as_set))
}

/// The left submodule is contained in the join.
theorem submodule_subset_sup_left[R: Ring, M: AddCommGroup](
    a: Submodule[R, M], b: Submodule[R, M]
) {
    submodule_subset(a, submodule_sup(a, b))
} by {
    forall(x: M) {
        if a.contains(x) {
            submodule_as_set_contains_eq(a, x)
            a.as_set.contains(x)
            union_contains_left(a.as_set, b.as_set, x)
            a.as_set.union(b.as_set).contains(x)
            submodule_closure_contains_of_set_contains(a.carrier, a.as_set.union(b.as_set), x)
            submodule_closure(a.carrier, a.as_set.union(b.as_set)).contains(x)
            submodule_sup(a, b).contains(x)
        }
    }
}

/// The right submodule is contained in the join.
theorem submodule_subset_sup_right[R: Ring, M: AddCommGroup](
    a: Submodule[R, M], b: Submodule[R, M]
) {
    submodule_subset(b, submodule_sup(a, b))
} by {
    forall(x: M) {
        if b.contains(x) {
            submodule_as_set_contains_eq(b, x)
            b.as_set.contains(x)
            union_contains_right(a.as_set, b.as_set, x)
            a.as_set.union(b.as_set).contains(x)
            submodule_closure_contains_of_set_contains(a.carrier, a.as_set.union(b.as_set), x)
            submodule_closure(a.carrier, a.as_set.union(b.as_set)).contains(x)
            submodule_sup(a, b).contains(x)
        }
    }
}

/// The join is contained in every common upper bound with the same ambient module.
theorem submodule_sup_subset_of_subset_left_right[R: Ring, M: AddCommGroup](
    a: Submodule[R, M],
    b: Submodule[R, M],
    c: Submodule[R, M]
) {
    c.carrier = a.carrier and submodule_subset(a, c) and submodule_subset(b, c) implies
        submodule_subset(submodule_sup(a, b), c)
} by {
    if c.carrier = a.carrier and submodule_subset(a, c) and submodule_subset(b, c) {
        forall(x: M) {
            if a.as_set.union(b.as_set).contains(x) {
                union_contains_eq(a.as_set, b.as_set, x)
                if a.as_set.contains(x) {
                    submodule_as_set_contains_eq(a, x)
                    a.contains(x)
                    c.contains(x)
                } else {
                    b.as_set.contains(x)
                    submodule_as_set_contains_eq(b, x)
                    b.contains(x)
                    c.contains(x)
                }
            }
        }
        set_subset_submodule(a.as_set.union(b.as_set), c) = forall(y: M) {
            a.as_set.union(b.as_set).contains(y) implies c.contains(y)
        }
        set_subset_submodule(a.as_set.union(b.as_set), c)
        submodule_closure_subset_of_set_subset(a.carrier, a.as_set.union(b.as_set), c)
        submodule_subset(submodule_closure(a.carrier, a.as_set.union(b.as_set)), c)
        submodule_subset(submodule_sup(a, b), c)
    }
}

/// Containment of a join is equivalent to containment of both submodules.
theorem submodule_sup_subset_iff[R: Ring, M: AddCommGroup](
    a: Submodule[R, M],
    b: Submodule[R, M],
    c: Submodule[R, M]
) {
    c.carrier = a.carrier implies
        (submodule_subset(submodule_sup(a, b), c) =
            (submodule_subset(a, c) and submodule_subset(b, c)))
} by {
    if c.carrier = a.carrier {
        if submodule_subset(submodule_sup(a, b), c) {
            submodule_subset_sup_left(a, b)
            submodule_subset_trans(a, submodule_sup(a, b), c)
            submodule_subset(a, c)
            submodule_subset_sup_right(a, b)
            submodule_subset_trans(b, submodule_sup(a, b), c)
            submodule_subset(b, c)
            submodule_subset(a, c) and submodule_subset(b, c)
        }
        if submodule_subset(a, c) and submodule_subset(b, c) {
            submodule_sup_subset_of_subset_left_right(a, b, c)
            submodule_subset(submodule_sup(a, b), c)
        }
        submodule_subset(submodule_sup(a, b), c) =
            (submodule_subset(a, c) and submodule_subset(b, c))
    }
}

/// The join has the same ambient module as the left submodule.
theorem submodule_sup_carrier_eq[R: Ring, M: AddCommGroup](
    a: Submodule[R, M], b: Submodule[R, M]
) {
    submodule_sup(a, b).carrier = a.carrier
} by {
    submodule_closure_carrier_eq(a.carrier, a.as_set.union(b.as_set))
    submodule_sup(a, b).carrier = a.carrier
}

/// Submodule join is commutative for same-carrier submodules.
theorem submodule_sup_comm[R: Ring, M: AddCommGroup](
    a: Submodule[R, M], b: Submodule[R, M]
) {
    a.carrier = b.carrier implies submodule_sup(a, b) = submodule_sup(b, a)
} by {
    if a.carrier = b.carrier {
        submodule_sup_carrier_eq(a, b)
        submodule_sup_carrier_eq(b, a)
        submodule_sup(a, b).carrier = submodule_sup(b, a).carrier

        submodule_subset_sup_right(b, a)
        submodule_subset(a, submodule_sup(b, a))
        submodule_subset_sup_left(b, a)
        submodule_subset(b, submodule_sup(b, a))
        submodule_sup_subset_of_subset_left_right(a, b, submodule_sup(b, a))
        submodule_subset(submodule_sup(a, b), submodule_sup(b, a))

        submodule_subset_sup_right(a, b)
        submodule_subset(b, submodule_sup(a, b))
        submodule_subset_sup_left(a, b)
        submodule_subset(a, submodule_sup(a, b))
        submodule_sup_subset_of_subset_left_right(b, a, submodule_sup(a, b))
        submodule_subset(submodule_sup(b, a), submodule_sup(a, b))

        submodule_subset_antisymm(submodule_sup(a, b), submodule_sup(b, a))
    }
}

/// Joining a submodule with itself gives the same submodule.
theorem submodule_sup_idempotent[R: Ring, M: AddCommGroup](a: Submodule[R, M]) {
    submodule_sup(a, a) = a
} by {
    submodule_subset_refl(a)
    submodule_subset_refl(a)
    submodule_sup_subset_of_subset_left_right(a, a, a)
    submodule_subset(submodule_sup(a, a), a)
    submodule_subset_sup_left(a, a)
    submodule_sup_carrier_eq(a, a)
    submodule_subset_antisymm(submodule_sup(a, a), a)
}

/// Joining with a larger same-carrier submodule on the right gives the larger submodule.
theorem submodule_sup_eq_right_of_subset[R: Ring, M: AddCommGroup](
    a: Submodule[R, M], b: Submodule[R, M]
) {
    a.carrier = b.carrier and submodule_subset(a, b) implies submodule_sup(a, b) = b
} by {
    if a.carrier = b.carrier and submodule_subset(a, b) {
        b.carrier = a.carrier
        submodule_subset_refl(b)
        submodule_sup_subset_of_subset_left_right(a, b, b)
        submodule_subset(submodule_sup(a, b), b)
        submodule_subset_sup_right(a, b)
        submodule_sup_carrier_eq(a, b)
        submodule_sup(a, b).carrier = b.carrier
        submodule_subset_antisymm(submodule_sup(a, b), b)
        submodule_sup(a, b) = b
    }
}

/// Joining with a larger same-carrier submodule on the left gives the larger submodule.
theorem submodule_sup_eq_left_of_subset[R: Ring, M: AddCommGroup](
    a: Submodule[R, M], b: Submodule[R, M]
) {
    a.carrier = b.carrier and submodule_subset(b, a) implies submodule_sup(a, b) = a
} by {
    if a.carrier = b.carrier and submodule_subset(b, a) {
        submodule_subset_refl(a)
        submodule_sup_subset_of_subset_left_right(a, b, a)
        submodule_subset(submodule_sup(a, b), a)
        submodule_subset_sup_left(a, b)
        submodule_sup_carrier_eq(a, b)
        submodule_sup(a, b).carrier = a.carrier
        submodule_subset_antisymm(submodule_sup(a, b), a)
        submodule_sup(a, b) = a
    }
}

/// The zero submodule is a left identity for join.
theorem zero_submodule_sup[R: Ring, M: AddCommGroup](carrier: Module[R, M], s: Submodule[R, M]) {
    s.carrier = carrier implies submodule_sup(zero_submodule(carrier), s) = s
} by {
    if s.carrier = carrier {
        zero_submodule_carrier_eq(carrier)
        zero_submodule(carrier).carrier = s.carrier
        zero_submodule_subset(s)
        submodule_sup_eq_right_of_subset(zero_submodule(carrier), s)
        submodule_sup(zero_submodule(carrier), s) = s
    }
}

/// The zero submodule is a right identity for join.
theorem submodule_sup_zero[R: Ring, M: AddCommGroup](s: Submodule[R, M]) {
    submodule_sup(s, zero_submodule(s.carrier)) = s
} by {
    zero_submodule_subset(s)
    zero_submodule_carrier_eq(s.carrier)
    s.carrier = zero_submodule(s.carrier).carrier
    submodule_sup_eq_left_of_subset(s, zero_submodule(s.carrier))
    submodule_sup(s, zero_submodule(s.carrier)) = s
}

/// The full submodule is a left absorbing element for join.
theorem full_submodule_sup[R: Ring, M: AddCommGroup](carrier: Module[R, M], s: Submodule[R, M]) {
    s.carrier = carrier implies submodule_sup(full_submodule(carrier), s) = full_submodule(carrier)
} by {
    if s.carrier = carrier {
        full_submodule_carrier_eq(carrier)
        full_submodule(carrier).carrier = s.carrier
        submodule_subset_full(s)
        submodule_sup_eq_left_of_subset(full_submodule(carrier), s)
        submodule_sup(full_submodule(carrier), s) = full_submodule(carrier)
    }
}

/// The full submodule is a right absorbing element for join.
theorem submodule_sup_full[R: Ring, M: AddCommGroup](s: Submodule[R, M]) {
    submodule_sup(s, full_submodule(s.carrier)) = full_submodule(s.carrier)
} by {
    submodule_subset_full(s)
    full_submodule_carrier_eq(s.carrier)
    s.carrier = full_submodule(s.carrier).carrier
    submodule_sup_eq_right_of_subset(s, full_submodule(s.carrier))
    submodule_sup(s, full_submodule(s.carrier)) = full_submodule(s.carrier)
}

attributes Submodule[R: Ring, M: AddCommGroup] {
    /// The smallest submodule containing a set.
    let closure: (Module[R, M], Set[M]) -> Submodule[R, M] = submodule_closure

    /// The least submodule containing this submodule and another submodule.
    let sup: (Submodule[R, M], Submodule[R, M]) -> Submodule[R, M] = submodule_sup
}

/// True if a submodule is generated by a finite set.
define submodule_is_finitely_generated[R: Ring, M: AddCommGroup](s: Submodule[R, M]) -> Bool {
    exists(a: Set[M]) {
        a.is_finite and submodule_closure(s.carrier, a) = s
    }
}

/// A finitely generated submodule has a finite generating set.
theorem submodule_is_finitely_generated_witness[R: Ring, M: AddCommGroup](
    s: Submodule[R, M]
) {
    submodule_is_finitely_generated(s) implies exists(a: Set[M]) {
        a.is_finite and submodule_closure(s.carrier, a) = s
    }
}

/// A submodule equal to the closure of a finite set is finitely generated.
theorem submodule_is_finitely_generated_of_closure_eq[R: Ring, M: AddCommGroup](
    carrier: Module[R, M], a: Set[M], s: Submodule[R, M]
) {
    a.is_finite and s.carrier = carrier and submodule_closure(carrier, a) = s
        implies submodule_is_finitely_generated(s)
} by {
    if a.is_finite and s.carrier = carrier and submodule_closure(carrier, a) = s {
        exists(b: Set[M]) {
            b.is_finite and submodule_closure(s.carrier, b) = s
        }
        submodule_is_finitely_generated(s)
    }
}

/// Equality preserves finite generation of submodules.
theorem submodule_is_finitely_generated_of_eq[R: Ring, M: AddCommGroup](
    s: Submodule[R, M], t: Submodule[R, M]
) {
    submodule_is_finitely_generated(s) and s = t implies submodule_is_finitely_generated(t)
} by {
    if submodule_is_finitely_generated(s) and s = t {
        submodule_is_finitely_generated(s) = exists(a: Set[M]) {
            a.is_finite and submodule_closure(s.carrier, a) = s
        }
        let a: Set[M] satisfy {
            a.is_finite and submodule_closure(s.carrier, a) = s
        }
        s.carrier = t.carrier
        submodule_closure(t.carrier, a) = t
        exists(b: Set[M]) {
            b.is_finite and submodule_closure(t.carrier, b) = t
        }
        submodule_is_finitely_generated(t)
    }
}

/// The closure of a finite set is a finitely generated submodule.
theorem submodule_closure_is_finitely_generated[R: Ring, M: AddCommGroup](
    carrier: Module[R, M], a: Set[M]
) {
    a.is_finite implies submodule_is_finitely_generated(submodule_closure(carrier, a))
} by {
    if a.is_finite {
        submodule_closure_carrier_eq(carrier, a)
        submodule_is_finitely_generated_of_closure_eq(carrier, a, submodule_closure(carrier, a))
    }
}

/// A submodule with finite underlying set is finitely generated.
theorem submodule_is_finitely_generated_of_as_set_finite[R: Ring, M: AddCommGroup](
    s: Submodule[R, M]
) {
    s.as_set.is_finite implies submodule_is_finitely_generated(s)
} by {
    if s.as_set.is_finite {
        submodule_closure_as_set(s)
        submodule_is_finitely_generated_of_closure_eq(s.carrier, s.as_set, s)
    }
}

/// The closure of one element is a finitely generated submodule.
theorem submodule_closure_singleton_is_finitely_generated[R: Ring, M: AddCommGroup](
    carrier: Module[R, M], a: M
) {
    submodule_is_finitely_generated(submodule_closure(carrier, Set[M].singleton(a)))
} by {
    singleton_set_is_finite(a)
    submodule_closure_is_finitely_generated(carrier, Set[M].singleton(a))
}

/// The generator belongs to the submodule generated by it.
theorem submodule_closure_singleton_contains[R: Ring, M: AddCommGroup](
    carrier: Module[R, M], a: M
) {
    submodule_closure(carrier, Set[M].singleton(a)).contains(a)
} by {
    singleton_contains_eq(a, a)
    submodule_closure_contains_of_set_contains(carrier, Set[M].singleton(a), a)
}

/// The zero submodule is finitely generated.
theorem zero_submodule_is_finitely_generated[R: Ring, M: AddCommGroup](carrier: Module[R, M]) {
    submodule_is_finitely_generated(zero_submodule(carrier))
} by {
    empty_set_is_finite[M]
    submodule_closure_is_finitely_generated(carrier, Set[M].empty_set)
    submodule_is_finitely_generated(submodule_closure(carrier, Set[M].empty_set))
    submodule_closure_empty(carrier)
    submodule_is_finitely_generated(zero_submodule(carrier))
}

/// The additive subgroup predicate associated to a submodule.
theorem submodule_add_subgroup_constraint[R: Ring, M: AddCommGroup](s: Submodule[R, M]) {
    add_subgroup_constraint(s.contains)
} by {
    submodule_contains_zero(s)
    add_zero_constraint(s.contains)
    forall(a: M, b: M) {
        if s.contains(a) and s.contains(b) {
            submodule_contains_add(s, a, b)
            s.contains(a + b)
        }
    }
    add_closure_constraint(s.contains)
    forall(a: M) {
        if s.contains(a) {
            submodule_contains_neg(s, a)
            s.contains(-a)
        }
    }
    neg_constraint(s.contains)
}

/// The additive subgroup associated to a submodule.
let submodule_to_add_subgroup[R: Ring, M: AddCommGroup](s: Submodule[R, M]) -> result: AddSubgroup[M] satisfy {
    AddSubgroup.new(s.contains) = Option.some(result)
} by {
    submodule_add_subgroup_constraint(s)
}

/// The additive subgroup associated to a submodule has the same membership predicate.
theorem submodule_to_add_subgroup_contains[R: Ring, M: AddCommGroup](s: Submodule[R, M]) {
    submodule_to_add_subgroup(s).contains = s.contains
}

/// Membership in the additive subgroup associated to a submodule is membership in the submodule.
theorem submodule_to_add_subgroup_contains_eq[R: Ring, M: AddCommGroup](s: Submodule[R, M], x: M) {
    submodule_to_add_subgroup(s).contains(x) = s.contains(x)
} by {
    submodule_to_add_subgroup_contains(s)
    submodule_to_add_subgroup(s).contains(x) = s.contains(x)
}

/// Membership in a submodule is membership in its associated additive subgroup.
theorem submodule_contains_to_add_subgroup_eq[R: Ring, M: AddCommGroup](s: Submodule[R, M], x: M) {
    s.contains(x) = submodule_to_add_subgroup(s).contains(x)
} by {
    submodule_to_add_subgroup_contains_eq(s, x)
    s.contains(x) = submodule_to_add_subgroup(s).contains(x)
}

/// The associated additive subgroup has the same underlying set as the submodule.
theorem submodule_to_add_subgroup_as_set[R: Ring, M: AddCommGroup](s: Submodule[R, M]) {
    submodule_to_add_subgroup(s).as_set = s.as_set
}

/// The additive submonoid predicate associated to a submodule.
theorem submodule_add_submonoid_constraint[R: Ring, M: AddCommGroup](s: Submodule[R, M]) {
    add_submonoid_constraint(s.contains)
} by {
    submodule_contains_zero(s)
    add_submonoid_zero_constraint(s.contains)
    forall(a: M, b: M) {
        if s.contains(a) and s.contains(b) {
            submodule_contains_add(s, a, b)
            s.contains(a + b)
        }
    }
    add_submonoid_closure_constraint(s.contains)
}

/// The additive submonoid associated to a submodule.
let submodule_to_add_submonoid[R: Ring, M: AddCommGroup](s: Submodule[R, M]) -> result: AddSubmonoid[M] satisfy {
    AddSubmonoid.new(s.contains) = Option.some(result)
} by {
    submodule_add_submonoid_constraint(s)
}

/// The additive submonoid associated to a submodule has the same membership predicate.
theorem submodule_to_add_submonoid_contains[R: Ring, M: AddCommGroup](s: Submodule[R, M]) {
    submodule_to_add_submonoid(s).contains = s.contains
}

/// Membership in the additive submonoid associated to a submodule is membership in the submodule.
theorem submodule_to_add_submonoid_contains_eq[R: Ring, M: AddCommGroup](s: Submodule[R, M], x: M) {
    submodule_to_add_submonoid(s).contains(x) = s.contains(x)
} by {
    submodule_to_add_submonoid_contains(s)
    submodule_to_add_submonoid(s).contains(x) = s.contains(x)
}

/// Membership in a submodule is membership in its associated additive submonoid.
theorem submodule_contains_to_add_submonoid_eq[R: Ring, M: AddCommGroup](s: Submodule[R, M], x: M) {
    s.contains(x) = submodule_to_add_submonoid(s).contains(x)
} by {
    submodule_to_add_submonoid_contains_eq(s, x)
    s.contains(x) = submodule_to_add_submonoid(s).contains(x)
}

/// The associated additive submonoid has the same underlying set as the submodule.
theorem submodule_to_add_submonoid_as_set[R: Ring, M: AddCommGroup](s: Submodule[R, M]) {
    submodule_to_add_submonoid(s).as_set = s.as_set
}

/// Membership in the kernel of a linear map.
define linear_map_kernel[R: Ring, M: AddCommGroup, N: AddCommGroup](
    f: M -> N, x: M
) -> Bool {
    f(x) = N.0
}

/// A linear map sends the additive identity to the additive identity.
theorem linear_map_zero[R: Ring, M: AddCommGroup, N: AddCommGroup](
    src: Module[R, M], dst: Module[R, N], f: M -> N
) {
    is_linear_map(src, dst, f) implies f(M.0) = N.0
} by {
    if is_linear_map(src, dst, f) {
        is_linear_map(src, dst, f) = preserves_add(f) and preserves_smul(src, dst, f)
        preserves_add(f)
        is_add_comm_group_hom(f)
        let g: AddCommGroupHom[M, N] satisfy {
            AddCommGroupHom[M, N].new(f) = Option.some(g)
        }
        add_comm_group_hom_zero(g)
        g.hom = f
        f(M.0) = N.0
    }
}

/// A linear map preserves additive negation.
theorem linear_map_neg[R: Ring, M: AddCommGroup, N: AddCommGroup](
    src: Module[R, M], dst: Module[R, N], f: M -> N, x: M
) {
    is_linear_map(src, dst, f) implies f(-x) = -f(x)
} by {
    if is_linear_map(src, dst, f) {
        is_linear_map(src, dst, f) = preserves_add(f) and preserves_smul(src, dst, f)
        preserves_add(f)
        is_add_comm_group_hom(f)
        let g: AddCommGroupHom[M, N] satisfy {
            AddCommGroupHom[M, N].new(f) = Option.some(g)
        }
        add_comm_group_hom_neg(g, x)
        g.hom = f
        f(-x) = -f(x)
    }
}

/// The kernel of a linear map contains the additive identity.
theorem linear_map_kernel_zero_constraint[R: Ring, M: AddCommGroup, N: AddCommGroup](
    src: Module[R, M], dst: Module[R, N], f: M -> N
) {
    is_linear_map(src, dst, f) implies submodule_zero_constraint(src, linear_map_kernel[R, M, N](f))
} by {
    if is_linear_map(src, dst, f) {
        linear_map_zero(src, dst, f)
        f(M.0) = N.0
        linear_map_kernel[R, M, N](f, M.0)
    }
}

/// The kernel of a linear map is closed under addition.
theorem linear_map_kernel_add_constraint[R: Ring, M: AddCommGroup, N: AddCommGroup](
    src: Module[R, M], dst: Module[R, N], f: M -> N
) {
    is_linear_map(src, dst, f) implies submodule_add_constraint(src, linear_map_kernel[R, M, N](f))
} by {
    if is_linear_map(src, dst, f) {
        forall(x: M, y: M) {
            if linear_map_kernel[R, M, N](f, x) and linear_map_kernel[R, M, N](f, y) {
                is_linear_map(src, dst, f) = preserves_add(f) and preserves_smul(src, dst, f)
                preserves_add(f)
                preserves_add(f) = forall(a: M, b: M) {
                    f(a + b) = f(a) + f(b)
                }
                f(x) = N.0
                f(y) = N.0
                f(x + y) = f(x) + f(y)
                f(x + y) = N.0 + N.0
                N.0 + N.0 = N.0
                f(x + y) = N.0
                linear_map_kernel[R, M, N](f, x + y)
            }
        }
    }
}

/// The kernel of a linear map is closed under additive negation.
theorem linear_map_kernel_neg_constraint[R: Ring, M: AddCommGroup, N: AddCommGroup](
    src: Module[R, M], dst: Module[R, N], f: M -> N
) {
    is_linear_map(src, dst, f) implies submodule_neg_constraint(src, linear_map_kernel[R, M, N](f))
} by {
    if is_linear_map(src, dst, f) {
        forall(x: M) {
            if linear_map_kernel[R, M, N](f, x) {
                f(x) = N.0
                linear_map_neg(src, dst, f, x)
                f(-x) = -f(x)
                -f(x) = -N.0
                -N.0 = N.0
                f(-x) = N.0
                linear_map_kernel[R, M, N](f, -x)
            }
        }
    }
}

/// The kernel of a linear map is closed under scalar multiplication.
theorem linear_map_kernel_smul_constraint[R: Ring, M: AddCommGroup, N: AddCommGroup](
    src: Module[R, M], dst: Module[R, N], f: M -> N
) {
    is_linear_map(src, dst, f) implies submodule_smul_constraint(src, linear_map_kernel[R, M, N](f))
} by {
    if is_linear_map(src, dst, f) {
        forall(r: R, x: M) {
            if linear_map_kernel[R, M, N](f, x) {
                is_linear_map(src, dst, f) = preserves_add(f) and preserves_smul(src, dst, f)
                preserves_smul(src, dst, f)
                preserves_smul(src, dst, f) = forall(q: R, a: M) {
                    f(src.smul(q, a)) = dst.smul(q, f(a))
                }
                f(x) = N.0
                f(src.smul(r, x)) = dst.smul(r, f(x))
                dst.smul(r, f(x)) = dst.smul(r, N.0)
                module_smul_zero_right(dst, r)
                dst.smul(r, N.0) = N.0
                f(src.smul(r, x)) = N.0
                linear_map_kernel[R, M, N](f, src.smul(r, x))
            }
        }
    }
}

/// The kernel predicate when the map is linear, and the full predicate otherwise.
define linear_map_kernel_submodule_contains[R: Ring, M: AddCommGroup, N: AddCommGroup](
    src: Module[R, M], dst: Module[R, N], f: M -> N, x: M
) -> Bool {
    if is_linear_map(src, dst, f) {
        linear_map_kernel[R, M, N](f, x)
    } else {
        full_submodule_contains[M](x)
    }
}

/// The kernel-submodule predicate agrees with the kernel predicate for a linear map.
theorem linear_map_kernel_submodule_contains_eq_of_linear[R: Ring, M: AddCommGroup, N: AddCommGroup](
    src: Module[R, M], dst: Module[R, N], f: M -> N, x: M
) {
    is_linear_map(src, dst, f) implies
        linear_map_kernel_submodule_contains(src, dst, f, x) = linear_map_kernel[R, M, N](f, x)
} by {
    if is_linear_map(src, dst, f) {
        linear_map_kernel_submodule_contains(src, dst, f, x) = linear_map_kernel[R, M, N](f, x)
    }
}

/// The kernel-submodule predicate is universal when the map is not known to be linear.
theorem linear_map_kernel_submodule_contains_eq_full_of_not_linear[R: Ring, M: AddCommGroup, N: AddCommGroup](
    src: Module[R, M], dst: Module[R, N], f: M -> N, x: M
) {
    not(is_linear_map(src, dst, f)) implies
        linear_map_kernel_submodule_contains(src, dst, f, x) = full_submodule_contains[M](x)
} by {
    if not(is_linear_map(src, dst, f)) {
        linear_map_kernel_submodule_contains(src, dst, f, x) = full_submodule_contains[M](x)
    }
}

/// The total kernel-submodule predicate is a submodule predicate.
theorem linear_map_kernel_submodule_is_submodule[R: Ring, M: AddCommGroup, N: AddCommGroup](
    src: Module[R, M], dst: Module[R, N], f: M -> N
) {
    is_submodule(src, linear_map_kernel_submodule_contains(src, dst, f))
} by {
    let p = linear_map_kernel_submodule_contains(src, dst, f)
    is_submodule(src, p) =
        (submodule_zero_constraint(src, p)
         and submodule_add_constraint(src, p)
         and submodule_neg_constraint(src, p)
         and submodule_smul_constraint(src, p))
    if is_linear_map(src, dst, f) {
        linear_map_kernel_zero_constraint(src, dst, f)
        submodule_zero_constraint(src, linear_map_kernel[R, M, N](f))
        linear_map_kernel_submodule_contains_eq_of_linear(src, dst, f, M.0)
        linear_map_kernel_submodule_contains(src, dst, f, M.0)
        submodule_zero_constraint(src, linear_map_kernel_submodule_contains(src, dst, f))

        linear_map_kernel_add_constraint(src, dst, f)
        submodule_add_constraint(src, linear_map_kernel[R, M, N](f)) = forall(a: M, b: M) {
            linear_map_kernel[R, M, N](f, a) and linear_map_kernel[R, M, N](f, b)
                implies linear_map_kernel[R, M, N](f, a + b)
        }
        forall(x: M) {
            forall(y: M) {
                if linear_map_kernel_submodule_contains(src, dst, f, x)
                    and linear_map_kernel_submodule_contains(src, dst, f, y) {
                    linear_map_kernel_submodule_contains_eq_of_linear(src, dst, f, x)
                    linear_map_kernel_submodule_contains_eq_of_linear(src, dst, f, y)
                    linear_map_kernel[R, M, N](f, x)
                    linear_map_kernel[R, M, N](f, y)
                    submodule_add_constraint(src, linear_map_kernel[R, M, N](f))
                    linear_map_kernel[R, M, N](f, x + y)
                    linear_map_kernel_submodule_contains_eq_of_linear(src, dst, f, x + y)
                    linear_map_kernel_submodule_contains(src, dst, f, x + y)
                }
            }
        }
        submodule_add_constraint(src, linear_map_kernel_submodule_contains(src, dst, f))

        linear_map_kernel_neg_constraint(src, dst, f)
        submodule_neg_constraint(src, linear_map_kernel[R, M, N](f)) = forall(a: M) {
            linear_map_kernel[R, M, N](f, a) implies linear_map_kernel[R, M, N](f, -a)
        }
        forall(x: M) {
            if linear_map_kernel_submodule_contains(src, dst, f, x) {
                linear_map_kernel_submodule_contains_eq_of_linear(src, dst, f, x)
                linear_map_kernel[R, M, N](f, x)
                submodule_neg_constraint(src, linear_map_kernel[R, M, N](f))
                linear_map_kernel[R, M, N](f, -x)
                linear_map_kernel_submodule_contains_eq_of_linear(src, dst, f, -x)
                linear_map_kernel_submodule_contains(src, dst, f, -x)
            }
        }
        submodule_neg_constraint(src, linear_map_kernel_submodule_contains(src, dst, f))

        linear_map_kernel_smul_constraint(src, dst, f)
        submodule_smul_constraint(src, linear_map_kernel[R, M, N](f)) = forall(q: R, a: M) {
            linear_map_kernel[R, M, N](f, a) implies linear_map_kernel[R, M, N](f, src.smul(q, a))
        }
        forall(r: R, x: M) {
            if linear_map_kernel_submodule_contains(src, dst, f, x) {
                linear_map_kernel_submodule_contains_eq_of_linear(src, dst, f, x)
                linear_map_kernel[R, M, N](f, x)
                submodule_smul_constraint(src, linear_map_kernel[R, M, N](f))
                linear_map_kernel[R, M, N](f, src.smul(r, x))
                linear_map_kernel_submodule_contains_eq_of_linear(src, dst, f, src.smul(r, x))
                linear_map_kernel_submodule_contains(src, dst, f, src.smul(r, x))
            }
        }
        submodule_smul_constraint(src, linear_map_kernel_submodule_contains(src, dst, f))
        is_submodule(src, p)
    } else {
        full_submodule_is_submodule(src)
        forall(x: M) {
            linear_map_kernel_submodule_contains_eq_full_of_not_linear(src, dst, f, x)
            linear_map_kernel_submodule_contains(src, dst, f, x)
        }
        submodule_zero_constraint(src, linear_map_kernel_submodule_contains(src, dst, f))
        forall(x: M, y: M) {
            if linear_map_kernel_submodule_contains(src, dst, f, x)
                and linear_map_kernel_submodule_contains(src, dst, f, y) {
                linear_map_kernel_submodule_contains(src, dst, f, x + y)
            }
        }
        submodule_add_constraint(src, linear_map_kernel_submodule_contains(src, dst, f))
        forall(x: M) {
            if linear_map_kernel_submodule_contains(src, dst, f, x) {
                linear_map_kernel_submodule_contains(src, dst, f, -x)
            }
        }
        submodule_neg_constraint(src, linear_map_kernel_submodule_contains(src, dst, f))
        forall(r: R, x: M) {
            if linear_map_kernel_submodule_contains(src, dst, f, x) {
                linear_map_kernel_submodule_contains(src, dst, f, src.smul(r, x))
            }
        }
        submodule_smul_constraint(src, linear_map_kernel_submodule_contains(src, dst, f))
        is_submodule(src, linear_map_kernel_submodule_contains(src, dst, f))
    }
}

/// The kernel submodule of a linear map, using the full submodule when the map is not linear.
let linear_map_kernel_submodule[R: Ring, M: AddCommGroup, N: AddCommGroup](src: Module[R, M], dst: Module[R, N], f: M -> N) -> result: Submodule[R, M] satisfy {
    Submodule.new(src, linear_map_kernel_submodule_contains(src, dst, f)) = Option.some(result)
} by {
    linear_map_kernel_submodule_is_submodule(src, dst, f)
}

/// The kernel submodule has the source module as its ambient module.
theorem linear_map_kernel_submodule_carrier_eq[R: Ring, M: AddCommGroup, N: AddCommGroup](
    src: Module[R, M], dst: Module[R, N], f: M -> N
) {
    linear_map_kernel_submodule(src, dst, f).carrier = src
}

/// Membership in the kernel submodule is equality to zero for a linear map.
theorem linear_map_kernel_submodule_contains_eq[R: Ring, M: AddCommGroup, N: AddCommGroup](
    src: Module[R, M], dst: Module[R, N], f: M -> N, x: M
) {
    is_linear_map(src, dst, f) implies
        linear_map_kernel_submodule(src, dst, f).contains(x) = (f(x) = N.0)
} by {
    if is_linear_map(src, dst, f) {
        let p = linear_map_kernel_submodule_contains(src, dst, f)
        linear_map_kernel_submodule_is_submodule(src, dst, f)
        is_submodule(src, p)
        let s: Submodule[R, M] satisfy {
            Submodule.new(src, p) = Option.some(s)
        }
        linear_map_kernel_submodule(src, dst, f) = s
        s.contains(x) = p(x)
        p(x) = linear_map_kernel_submodule_contains(src, dst, f, x)
        linear_map_kernel_submodule(src, dst, f).contains(x) = linear_map_kernel_submodule_contains(src, dst, f, x)
        linear_map_kernel_submodule_contains_eq_of_linear(src, dst, f, x)
        linear_map_kernel_submodule_contains(src, dst, f, x) = linear_map_kernel[R, M, N](f, x)
        linear_map_kernel[R, M, N](f, x) = (f(x) = N.0)
        linear_map_kernel_submodule(src, dst, f).contains(x) = (f(x) = N.0)
    }
}

/// An element whose image is zero belongs to the kernel submodule of a linear map.
theorem linear_map_kernel_submodule_contains_of_map_eq_zero[R: Ring, M: AddCommGroup, N: AddCommGroup](
    src: Module[R, M], dst: Module[R, N], f: M -> N, x: M
) {
    is_linear_map(src, dst, f) and f(x) = N.0
    implies linear_map_kernel_submodule(src, dst, f).contains(x)
} by {
    if is_linear_map(src, dst, f) and f(x) = N.0 {
        linear_map_kernel_submodule_contains_eq(src, dst, f, x)
        linear_map_kernel_submodule(src, dst, f).contains(x) = (f(x) = N.0)
        linear_map_kernel_submodule(src, dst, f).contains(x)
    }
}

/// A member of the kernel submodule of a linear map has image zero.
theorem linear_map_kernel_submodule_map_eq_zero_of_contains[R: Ring, M: AddCommGroup, N: AddCommGroup](
    src: Module[R, M], dst: Module[R, N], f: M -> N, x: M
) {
    is_linear_map(src, dst, f) and linear_map_kernel_submodule(src, dst, f).contains(x)
    implies f(x) = N.0
} by {
    if is_linear_map(src, dst, f) and linear_map_kernel_submodule(src, dst, f).contains(x) {
        linear_map_kernel_submodule_contains_eq(src, dst, f, x)
        linear_map_kernel_submodule(src, dst, f).contains(x) = (f(x) = N.0)
        f(x) = N.0
    }
}

/// A submodule is contained in the kernel precisely when all of its elements map to zero.
theorem linear_map_kernel_submodule_subset_iff[R: Ring, M: AddCommGroup, N: AddCommGroup](
    src: Module[R, M], dst: Module[R, N], f: M -> N, s: Submodule[R, M]
) {
    is_linear_map(src, dst, f) implies
        (submodule_subset(s, linear_map_kernel_submodule(src, dst, f)) =
            forall(x: M) { s.contains(x) implies f(x) = N.0 })
} by {
    if is_linear_map(src, dst, f) {
        if submodule_subset(s, linear_map_kernel_submodule(src, dst, f)) {
            submodule_subset(s, linear_map_kernel_submodule(src, dst, f)) = forall(y: M) {
                s.contains(y) implies linear_map_kernel_submodule(src, dst, f).contains(y)
            }
            forall(x: M) {
                if s.contains(x) {
                    linear_map_kernel_submodule(src, dst, f).contains(x)
                    linear_map_kernel_submodule_map_eq_zero_of_contains(src, dst, f, x)
                    f(x) = N.0
                }
            }
        }
        if forall(x: M) { s.contains(x) implies f(x) = N.0 } {
            forall(x: M) {
                if s.contains(x) {
                    f(x) = N.0
                    linear_map_kernel_submodule_contains_of_map_eq_zero(src, dst, f, x)
                    linear_map_kernel_submodule(src, dst, f).contains(x)
                }
            }
            submodule_subset(s, linear_map_kernel_submodule(src, dst, f))
        }
        submodule_subset(s, linear_map_kernel_submodule(src, dst, f)) =
            forall(x: M) { s.contains(x) implies f(x) = N.0 }
    }
}

/// Membership in the image of a linear map.
define linear_map_image[M: AddCommGroup, N: AddCommGroup](f: M -> N, y: N) -> Bool {
    exists(x: M) {
        f(x) = y
    }
}

/// The image of a linear map contains the additive identity.
theorem linear_map_image_zero_constraint[R: Ring, M: AddCommGroup, N: AddCommGroup](
    src: Module[R, M], dst: Module[R, N], f: M -> N
) {
    is_linear_map(src, dst, f) implies submodule_zero_constraint(dst, linear_map_image[M, N](f))
} by {
    if is_linear_map(src, dst, f) {
        linear_map_zero(src, dst, f)
        f(M.0) = N.0
        exists(x: M) {
            f(x) = N.0
        }
        linear_map_image[M, N](f, N.0)
    }
}

/// The image of a linear map is closed under addition.
theorem linear_map_image_add_constraint[R: Ring, M: AddCommGroup, N: AddCommGroup](
    src: Module[R, M], dst: Module[R, N], f: M -> N
) {
    is_linear_map(src, dst, f) implies submodule_add_constraint(dst, linear_map_image[M, N](f))
} by {
    if is_linear_map(src, dst, f) {
        forall(y: N, z: N) {
            if linear_map_image[M, N](f, y) and linear_map_image[M, N](f, z) {
                let x: M satisfy { f(x) = y }
                let w: M satisfy { f(w) = z }
                is_linear_map(src, dst, f) = preserves_add(f) and preserves_smul(src, dst, f)
                preserves_add(f)
                preserves_add(f) = forall(a: M, b: M) {
                    f(a + b) = f(a) + f(b)
                }
                f(x + w) = f(x) + f(w)
                f(x) + f(w) = y + z
                f(x + w) = y + z
                exists(t: M) {
                    f(t) = y + z
                }
                linear_map_image[M, N](f, y + z)
            }
        }
    }
}

/// The image of a linear map is closed under additive negation.
theorem linear_map_image_neg_constraint[R: Ring, M: AddCommGroup, N: AddCommGroup](
    src: Module[R, M], dst: Module[R, N], f: M -> N
) {
    is_linear_map(src, dst, f) implies submodule_neg_constraint(dst, linear_map_image[M, N](f))
} by {
    if is_linear_map(src, dst, f) {
        forall(y: N) {
            if linear_map_image[M, N](f, y) {
                let x: M satisfy { f(x) = y }
                linear_map_neg(src, dst, f, x)
                f(-x) = -f(x)
                -f(x) = -y
                f(-x) = -y
                exists(t: M) {
                    f(t) = -y
                }
                linear_map_image[M, N](f, -y)
            }
        }
    }
}

/// The image of a linear map is closed under scalar multiplication.
theorem linear_map_image_smul_constraint[R: Ring, M: AddCommGroup, N: AddCommGroup](
    src: Module[R, M], dst: Module[R, N], f: M -> N
) {
    is_linear_map(src, dst, f) implies submodule_smul_constraint(dst, linear_map_image[M, N](f))
} by {
    if is_linear_map(src, dst, f) {
        forall(r: R, y: N) {
            if linear_map_image[M, N](f, y) {
                let x: M satisfy { f(x) = y }
                is_linear_map(src, dst, f) = preserves_add(f) and preserves_smul(src, dst, f)
                preserves_smul(src, dst, f)
                preserves_smul(src, dst, f) = forall(q: R, a: M) {
                    f(src.smul(q, a)) = dst.smul(q, f(a))
                }
                f(src.smul(r, x)) = dst.smul(r, f(x))
                dst.smul(r, f(x)) = dst.smul(r, y)
                f(src.smul(r, x)) = dst.smul(r, y)
                exists(t: M) {
                    f(t) = dst.smul(r, y)
                }
                linear_map_image[M, N](f, dst.smul(r, y))
            }
        }
    }
}

/// The image predicate when the map is linear, and the full predicate otherwise.
define linear_map_image_submodule_contains[R: Ring, M: AddCommGroup, N: AddCommGroup](
    src: Module[R, M], dst: Module[R, N], f: M -> N, y: N
) -> Bool {
    if is_linear_map(src, dst, f) {
        linear_map_image[M, N](f, y)
    } else {
        full_submodule_contains[N](y)
    }
}

/// The image-submodule predicate agrees with the image predicate for a linear map.
theorem linear_map_image_submodule_contains_eq_of_linear[R: Ring, M: AddCommGroup, N: AddCommGroup](
    src: Module[R, M], dst: Module[R, N], f: M -> N, y: N
) {
    is_linear_map(src, dst, f) implies
        linear_map_image_submodule_contains(src, dst, f, y) = linear_map_image[M, N](f, y)
} by {
    if is_linear_map(src, dst, f) {
        linear_map_image_submodule_contains(src, dst, f, y) = linear_map_image[M, N](f, y)
    }
}

/// The image-submodule predicate is universal when the map is not known to be linear.
theorem linear_map_image_submodule_contains_eq_full_of_not_linear[R: Ring, M: AddCommGroup, N: AddCommGroup](
    src: Module[R, M], dst: Module[R, N], f: M -> N, y: N
) {
    not(is_linear_map(src, dst, f)) implies
        linear_map_image_submodule_contains(src, dst, f, y) = full_submodule_contains[N](y)
} by {
    if not(is_linear_map(src, dst, f)) {
        linear_map_image_submodule_contains(src, dst, f, y) = full_submodule_contains[N](y)
    }
}

/// The total image-submodule predicate is a submodule predicate.
theorem linear_map_image_submodule_is_submodule[R: Ring, M: AddCommGroup, N: AddCommGroup](
    src: Module[R, M], dst: Module[R, N], f: M -> N
) {
    is_submodule(dst, linear_map_image_submodule_contains(src, dst, f))
} by {
    let p = linear_map_image_submodule_contains(src, dst, f)
    is_submodule(dst, p) =
        (submodule_zero_constraint(dst, p)
         and submodule_add_constraint(dst, p)
         and submodule_neg_constraint(dst, p)
         and submodule_smul_constraint(dst, p))
    if is_linear_map(src, dst, f) {
        linear_map_image_zero_constraint(src, dst, f)
        submodule_zero_constraint(dst, linear_map_image[M, N](f))
        linear_map_image_submodule_contains_eq_of_linear(src, dst, f, N.0)
        linear_map_image_submodule_contains(src, dst, f, N.0)
        submodule_zero_constraint(dst, linear_map_image_submodule_contains(src, dst, f))

        linear_map_image_add_constraint(src, dst, f)
        submodule_add_constraint(dst, linear_map_image[M, N](f)) = forall(a: N, b: N) {
            linear_map_image[M, N](f, a) and linear_map_image[M, N](f, b)
                implies linear_map_image[M, N](f, a + b)
        }
        forall(y: N) {
            forall(z: N) {
                if linear_map_image_submodule_contains(src, dst, f, y)
                    and linear_map_image_submodule_contains(src, dst, f, z) {
                    linear_map_image_submodule_contains_eq_of_linear(src, dst, f, y)
                    linear_map_image_submodule_contains_eq_of_linear(src, dst, f, z)
                    linear_map_image[M, N](f, y)
                    linear_map_image[M, N](f, z)
                    submodule_add_constraint(dst, linear_map_image[M, N](f))
                    linear_map_image[M, N](f, y + z)
                    linear_map_image_submodule_contains_eq_of_linear(src, dst, f, y + z)
                    linear_map_image_submodule_contains(src, dst, f, y + z)
                }
            }
        }
        submodule_add_constraint(dst, linear_map_image_submodule_contains(src, dst, f))

        linear_map_image_neg_constraint(src, dst, f)
        submodule_neg_constraint(dst, linear_map_image[M, N](f)) = forall(a: N) {
            linear_map_image[M, N](f, a) implies linear_map_image[M, N](f, -a)
        }
        forall(y: N) {
            if linear_map_image_submodule_contains(src, dst, f, y) {
                linear_map_image_submodule_contains_eq_of_linear(src, dst, f, y)
                linear_map_image[M, N](f, y)
                submodule_neg_constraint(dst, linear_map_image[M, N](f))
                linear_map_image[M, N](f, -y)
                linear_map_image_submodule_contains_eq_of_linear(src, dst, f, -y)
                linear_map_image_submodule_contains(src, dst, f, -y)
            }
        }
        submodule_neg_constraint(dst, linear_map_image_submodule_contains(src, dst, f))

        linear_map_image_smul_constraint(src, dst, f)
        submodule_smul_constraint(dst, linear_map_image[M, N](f)) = forall(q: R, a: N) {
            linear_map_image[M, N](f, a) implies linear_map_image[M, N](f, dst.smul(q, a))
        }
        forall(r: R, y: N) {
            if linear_map_image_submodule_contains(src, dst, f, y) {
                linear_map_image_submodule_contains_eq_of_linear(src, dst, f, y)
                linear_map_image[M, N](f, y)
                submodule_smul_constraint(dst, linear_map_image[M, N](f))
                linear_map_image[M, N](f, dst.smul(r, y))
                linear_map_image_submodule_contains_eq_of_linear(src, dst, f, dst.smul(r, y))
                linear_map_image_submodule_contains(src, dst, f, dst.smul(r, y))
            }
        }
        submodule_smul_constraint(dst, linear_map_image_submodule_contains(src, dst, f))
        is_submodule(dst, p)
    } else {
        full_submodule_is_submodule(dst)
        forall(y: N) {
            linear_map_image_submodule_contains_eq_full_of_not_linear(src, dst, f, y)
            linear_map_image_submodule_contains(src, dst, f, y)
        }
        submodule_zero_constraint(dst, linear_map_image_submodule_contains(src, dst, f))
        forall(y: N, z: N) {
            if linear_map_image_submodule_contains(src, dst, f, y)
                and linear_map_image_submodule_contains(src, dst, f, z) {
                linear_map_image_submodule_contains(src, dst, f, y + z)
            }
        }
        submodule_add_constraint(dst, linear_map_image_submodule_contains(src, dst, f))
        forall(y: N) {
            if linear_map_image_submodule_contains(src, dst, f, y) {
                linear_map_image_submodule_contains(src, dst, f, -y)
            }
        }
        submodule_neg_constraint(dst, linear_map_image_submodule_contains(src, dst, f))
        forall(r: R, y: N) {
            if linear_map_image_submodule_contains(src, dst, f, y) {
                linear_map_image_submodule_contains(src, dst, f, dst.smul(r, y))
            }
        }
        submodule_smul_constraint(dst, linear_map_image_submodule_contains(src, dst, f))
        is_submodule(dst, linear_map_image_submodule_contains(src, dst, f))
    }
}

/// The image submodule of a linear map, using the full submodule when the map is not linear.
let linear_map_image_submodule[R: Ring, M: AddCommGroup, N: AddCommGroup](src: Module[R, M], dst: Module[R, N], f: M -> N) -> result: Submodule[R, N] satisfy {
    Submodule.new(dst, linear_map_image_submodule_contains(src, dst, f)) = Option.some(result)
} by {
    linear_map_image_submodule_is_submodule(src, dst, f)
}

/// The image submodule has the destination module as its ambient module.
theorem linear_map_image_submodule_carrier_eq[R: Ring, M: AddCommGroup, N: AddCommGroup](
    src: Module[R, M], dst: Module[R, N], f: M -> N
) {
    linear_map_image_submodule(src, dst, f).carrier = dst
}

/// Membership in the image submodule is membership in the image for a linear map.
theorem linear_map_image_submodule_contains_eq[R: Ring, M: AddCommGroup, N: AddCommGroup](
    src: Module[R, M], dst: Module[R, N], f: M -> N, y: N
) {
    is_linear_map(src, dst, f) implies
        linear_map_image_submodule(src, dst, f).contains(y) = linear_map_image[M, N](f, y)
} by {
    if is_linear_map(src, dst, f) {
        let p = linear_map_image_submodule_contains(src, dst, f)
        linear_map_image_submodule_is_submodule(src, dst, f)
        is_submodule(dst, p)
        let s: Submodule[R, N] satisfy {
            Submodule.new(dst, p) = Option.some(s)
        }
        Submodule.new(dst, p) = Option.some(linear_map_image_submodule(src, dst, f))
        linear_map_image_submodule(src, dst, f) = s
        s.contains(y) = p(y)
        p(y) = linear_map_image_submodule_contains(src, dst, f, y)
        linear_map_image_submodule(src, dst, f).contains(y) = linear_map_image_submodule_contains(src, dst, f, y)
        linear_map_image_submodule_contains_eq_of_linear(src, dst, f, y)
        linear_map_image_submodule(src, dst, f).contains(y) = linear_map_image[M, N](f, y)
    }
}

/// The value of a linear map belongs to its image submodule.
theorem linear_map_image_submodule_contains_apply[R: Ring, M: AddCommGroup, N: AddCommGroup](
    src: Module[R, M], dst: Module[R, N], f: M -> N, x: M
) {
    is_linear_map(src, dst, f) implies linear_map_image_submodule(src, dst, f).contains(f(x))
} by {
    if is_linear_map(src, dst, f) {
        exists(t: M) {
            f(t) = f(x)
        }
        linear_map_image[M, N](f, f(x))
        linear_map_image_submodule_contains_eq(src, dst, f, f(x))
        linear_map_image_submodule(src, dst, f).contains(f(x))
    }
}

/// Membership in the image submodule gives membership in the image predicate for a linear map.
theorem linear_map_image_of_submodule_contains[R: Ring, M: AddCommGroup, N: AddCommGroup](
    src: Module[R, M], dst: Module[R, N], f: M -> N, y: N
) {
    is_linear_map(src, dst, f) and linear_map_image_submodule(src, dst, f).contains(y)
    implies linear_map_image[M, N](f, y)
} by {
    if is_linear_map(src, dst, f) and linear_map_image_submodule(src, dst, f).contains(y) {
        linear_map_image_submodule_contains_eq(src, dst, f, y)
        linear_map_image_submodule(src, dst, f).contains(y) = linear_map_image[M, N](f, y)
        linear_map_image[M, N](f, y)
    }
}

/// The image submodule is contained in a submodule precisely when every map value belongs to it.
theorem linear_map_image_submodule_subset_iff[R: Ring, M: AddCommGroup, N: AddCommGroup](
    src: Module[R, M], dst: Module[R, N], f: M -> N, s: Submodule[R, N]
) {
    is_linear_map(src, dst, f) implies
        (submodule_subset(linear_map_image_submodule(src, dst, f), s) =
            forall(x: M) { s.contains(f(x)) })
} by {
    if is_linear_map(src, dst, f) {
        if submodule_subset(linear_map_image_submodule(src, dst, f), s) {
            submodule_subset(linear_map_image_submodule(src, dst, f), s) = forall(y: N) {
                linear_map_image_submodule(src, dst, f).contains(y) implies s.contains(y)
            }
            forall(x: M) {
                linear_map_image_submodule_contains_apply(src, dst, f, x)
                linear_map_image_submodule(src, dst, f).contains(f(x))
                s.contains(f(x))
            }
        }
        if forall(x: M) { s.contains(f(x)) } {
            forall(y: N) {
                if linear_map_image_submodule(src, dst, f).contains(y) {
                    linear_map_image_of_submodule_contains(src, dst, f, y)
                    linear_map_image[M, N](f, y)
                    let x: M satisfy { f(x) = y }
                    s.contains(f(x))
                    s.contains(y)
                }
            }
            submodule_subset(linear_map_image_submodule(src, dst, f), s)
        }
        submodule_subset(linear_map_image_submodule(src, dst, f), s) =
            forall(x: M) { s.contains(f(x)) }
    }
}

/// True if every element in the kernel of a linear map is the additive identity.
define is_kernel_trivial[R: Ring, M: AddCommGroup, N: AddCommGroup](f: M -> N) -> Bool {
    forall(x: M) {
        linear_map_kernel[R, M, N](f, x) implies x = M.0
    }
}

/// An injective linear map has a trivial kernel.
theorem linear_map_kernel_trivial_of_injective[R: Ring, M: AddCommGroup, N: AddCommGroup](
    src: Module[R, M], dst: Module[R, N], f: M -> N
) {
    is_linear_map(src, dst, f) and is_injective_fn(f) implies is_kernel_trivial[R, M, N](f)
} by {
    if is_linear_map(src, dst, f) and is_injective_fn(f) {
        forall(x: M) {
            if linear_map_kernel[R, M, N](f, x) {
                f(x) = N.0
                linear_map_zero(src, dst, f)
                f(M.0) = N.0
                f(x) = f(M.0)
                is_injective_fn(f) = forall(a: M, b: M) {
                    f(a) = f(b) implies a = b
                }
                x = M.0
            }
        }
    }
}

/// A linear map with a trivial kernel is injective.
theorem linear_map_injective_of_kernel_trivial[R: Ring, M: AddCommGroup, N: AddCommGroup](
    src: Module[R, M], dst: Module[R, N], f: M -> N
) {
    is_linear_map(src, dst, f) and is_kernel_trivial[R, M, N](f) implies is_injective_fn(f)
} by {
    if is_linear_map(src, dst, f) and is_kernel_trivial[R, M, N](f) {
        forall(x: M, y: M) {
            if f(x) = f(y) {
                linear_map_neg(src, dst, f, y)
                f(-y) = -f(y)
                is_linear_map(src, dst, f) = preserves_add(f) and preserves_smul(src, dst, f)
                preserves_add(f)
                preserves_add(f) = forall(a: M, b: M) {
                    f(a + b) = f(a) + f(b)
                }
                f(x + (-y)) = f(x) + f(-y)
                f(x) + f(-y) = f(x) + (-f(y))
                f(x) + (-f(y)) = f(y) + (-f(y))
                f(y) + (-f(y)) = N.0
                f(x + (-y)) = N.0
                linear_map_kernel[R, M, N](f, x + (-y))
                is_kernel_trivial[R, M, N](f) = forall(a: M) {
                    linear_map_kernel[R, M, N](f, a) implies a = M.0
                }
                x + (-y) = M.0
                x + (-y) + y = M.0 + y
                x + (-y + y) = y
                -y + y = M.0
                x + M.0 = y
                x = y
            }
        }
    }
}

/// Membership in the kernel submodule equals membership in the zero submodule under a trivial-kernel linear map.
theorem linear_map_kernel_submodule_contains_eq_zero_of_trivial[R: Ring, M: AddCommGroup, N: AddCommGroup](
    src: Module[R, M], dst: Module[R, N], f: M -> N, x: M
) {
    is_linear_map(src, dst, f) and is_kernel_trivial[R, M, N](f) implies
        linear_map_kernel_submodule(src, dst, f).contains(x) = zero_submodule(src).contains(x)
} by {
    if is_linear_map(src, dst, f) and is_kernel_trivial[R, M, N](f) {
        linear_map_kernel_submodule_contains_eq(src, dst, f, x)
        zero_submodule_contains_eq(src, x)
        if linear_map_kernel_submodule(src, dst, f).contains(x) {
            f(x) = N.0
            linear_map_kernel[R, M, N](f, x)
            is_kernel_trivial[R, M, N](f) = forall(a: M) {
                linear_map_kernel[R, M, N](f, a) implies a = M.0
            }
            x = M.0
            zero_submodule(src).contains(x)
        }
        if zero_submodule(src).contains(x) {
            x = M.0
            linear_map_zero(src, dst, f)
            f(M.0) = N.0
            f(x) = N.0
            linear_map_kernel_submodule_contains_of_map_eq_zero(src, dst, f, x)
            linear_map_kernel_submodule(src, dst, f).contains(x)
        }
        linear_map_kernel_submodule(src, dst, f).contains(x) = zero_submodule(src).contains(x)
    }
}

/// The kernel submodule of a linear map with trivial kernel is contained in the zero submodule.
theorem linear_map_kernel_submodule_subset_zero_of_trivial[R: Ring, M: AddCommGroup, N: AddCommGroup](
    src: Module[R, M], dst: Module[R, N], f: M -> N
) {
    is_linear_map(src, dst, f) and is_kernel_trivial[R, M, N](f)
    implies submodule_subset(linear_map_kernel_submodule(src, dst, f), zero_submodule(src))
} by {
    if is_linear_map(src, dst, f) and is_kernel_trivial[R, M, N](f) {
        forall(x: M) {
            if linear_map_kernel_submodule(src, dst, f).contains(x) {
                linear_map_kernel_submodule_contains_eq_zero_of_trivial(src, dst, f, x)
                zero_submodule(src).contains(x)
            }
        }
    }
}


/// A linear map has a trivial kernel when its kernel submodule equals the zero submodule.
theorem linear_map_kernel_trivial_of_kernel_submodule_eq_zero[R: Ring, M: AddCommGroup, N: AddCommGroup](
    src: Module[R, M], dst: Module[R, N], f: M -> N
) {
    is_linear_map(src, dst, f) and linear_map_kernel_submodule(src, dst, f) = zero_submodule(src)
        implies is_kernel_trivial[R, M, N](f)
} by {
    if is_linear_map(src, dst, f) and linear_map_kernel_submodule(src, dst, f) = zero_submodule(src) {
        forall(x: M) {
            if linear_map_kernel[R, M, N](f, x) {
                f(x) = N.0
                linear_map_kernel_submodule_contains_of_map_eq_zero(src, dst, f, x)
                linear_map_kernel_submodule(src, dst, f).contains(x)
                zero_submodule(src).contains(x)
                zero_submodule_contains_eq(src, x)
                x = M.0
            }
        }
    }
}

/// The full submodule is contained in the image submodule of a surjective linear map.
theorem full_submodule_subset_linear_map_image_submodule_of_surjective[R: Ring, M: AddCommGroup, N: AddCommGroup](
    src: Module[R, M], dst: Module[R, N], f: M -> N
) {
    is_linear_map(src, dst, f) and is_surjective_fn(f)
    implies submodule_subset(full_submodule(dst), linear_map_image_submodule(src, dst, f))
} by {
    if is_linear_map(src, dst, f) and is_surjective_fn(f) {
        forall(y: N) {
            if full_submodule(dst).contains(y) {
                is_surjective_fn(f) = forall(z: N) {
                    exists(x: M) {
                        f(x) = z
                    }
                }
                exists(x: M) {
                    f(x) = y
                }
                linear_map_image[M, N](f, y)
                linear_map_image_submodule_contains_eq(src, dst, f, y)
                linear_map_image_submodule(src, dst, f).contains(y)
            }
        }
    }
}

/// A point of the target lies in the image predicate when the image submodule is full.
theorem linear_map_image_of_image_submodule_eq_full[R: Ring, M: AddCommGroup, N: AddCommGroup](
    src: Module[R, M], dst: Module[R, N], f: M -> N, y: N
) {
    is_linear_map(src, dst, f) and linear_map_image_submodule(src, dst, f) = full_submodule(dst)
    implies linear_map_image[M, N](f, y)
} by {
    if is_linear_map(src, dst, f) and linear_map_image_submodule(src, dst, f) = full_submodule(dst) {
        full_submodule_contains_everything(dst, y)
        linear_map_image_submodule(src, dst, f).contains = full_submodule(dst).contains
        linear_map_image_submodule(src, dst, f).contains(y)
        linear_map_image_of_submodule_contains(src, dst, f, y)
        linear_map_image[M, N](f, y)
    }
}


/// The equivalence relation on a module defined by a submodule: two elements are
/// related when their difference lies in the submodule.
define submodule_rel[R: Ring, M: AddCommGroup](s: Submodule[R, M], a: M, b: M) -> Bool {
    s.contains(a - b)
}

/// The submodule quotient relation agrees with the additive-subgroup quotient relation pointwise.
theorem submodule_rel_eq_add_subgroup_rel[R: Ring, M: AddCommGroup](s: Submodule[R, M], a: M, b: M) {
    submodule_rel(s, a, b) = add_subgroup_rel(submodule_to_add_subgroup(s), a, b)
} by {
    submodule_to_add_subgroup_contains_eq(s, a - b)
}

/// The submodule relation as a function equals the additive-subgroup relation as a function.
theorem submodule_rel_eq_add_subgroup_rel_fn[R: Ring, M: AddCommGroup](s: Submodule[R, M]) {
    submodule_rel(s) = add_subgroup_rel(submodule_to_add_subgroup(s))
} by {
    forall(a: M) {
        forall(b: M) {
            submodule_rel_eq_add_subgroup_rel(s, a, b)
        }
        submodule_rel(s, a) = add_subgroup_rel(submodule_to_add_subgroup(s), a)
    }
}

/// The quotient relation associated with a submodule.
let submodule_quotient_relation[R: Ring, M: AddCommGroup](s: Submodule[R, M]) -> result: QuotientRelation[M] satisfy {
    result = add_subgroup_quotient_relation(submodule_to_add_subgroup(s))
}

/// The submodule quotient relation has the expected underlying relation.
theorem submodule_quotient_relation_rel[R: Ring, M: AddCommGroup](s: Submodule[R, M]) {
    submodule_quotient_relation(s).rel = submodule_rel(s)
} by {
    submodule_quotient_relation(s) = add_subgroup_quotient_relation(submodule_to_add_subgroup(s))
    add_subgroup_quotient_relation_rel(submodule_to_add_subgroup(s))
    submodule_rel_eq_add_subgroup_rel_fn(s)
}

/// In a module, scalar multiplication distributes over subtraction.
theorem module_smul_sub[R: Ring, M: AddCommGroup](m: Module[R, M], r: R, x: M, y: M) {
    m.smul(r, x - y) = m.smul(r, x) - m.smul(r, y)
} by {
    x - y = x + -y
    module_smul_add_right(m, r, x, -y)
    m.smul(r, x + -y) = m.smul(r, x) + m.smul(r, -y)
    module_smul_neg_right(m, r, y)
    m.smul(r, -y) = -m.smul(r, y)
    m.smul(r, x) + m.smul(r, -y) = m.smul(r, x) + -m.smul(r, y)
    m.smul(r, x) + -m.smul(r, y) = m.smul(r, x) - m.smul(r, y)
}

/// The submodule relation is preserved by left scalar multiplication on related elements.
theorem submodule_rel_smul_step[R: Ring, M: AddCommGroup](s: Submodule[R, M], r: R, a: M, b: M) {
    submodule_rel(s, a, b) implies submodule_rel(s, s.carrier.smul(r, a), s.carrier.smul(r, b))
} by {
    if submodule_rel(s, a, b) {
        s.contains(a - b)
        submodule_contains_smul(s, r, a - b)
        s.contains(s.carrier.smul(r, a - b))
        module_smul_sub(s.carrier, r, a, b)
        s.carrier.smul(r, a - b) = s.carrier.smul(r, a) - s.carrier.smul(r, b)
        s.contains(s.carrier.smul(r, a) - s.carrier.smul(r, b))
    }
}

/// Left scalar multiplication by a fixed scalar, viewed as a unary operation on the module.
define submodule_smul_by[R: Ring, M: AddCommGroup](s: Submodule[R, M], r: R) -> M -> M {
    function(x: M) {
        s.carrier.smul(r, x)
    }
}

/// The submodule relation is compatible with left scalar multiplication by a fixed scalar.
theorem submodule_rel_respects_smul_by[R: Ring, M: AddCommGroup](s: Submodule[R, M], r: R) {
    respects_unary_op(submodule_rel(s), submodule_smul_by(s, r))
} by {
    forall(a: M, b: M) {
        if submodule_rel(s, a, b) {
            submodule_rel_smul_step(s, r, a, b)
            submodule_rel(s, s.carrier.smul(r, a), s.carrier.smul(r, b))
            submodule_smul_by(s, r, a) = s.carrier.smul(r, a)
            submodule_smul_by(s, r, b) = s.carrier.smul(r, b)
            submodule_rel(s, submodule_smul_by(s, r, a), submodule_smul_by(s, r, b))
        }
    }
}

/// The submodule quotient relation respects left scalar multiplication by a fixed scalar.
theorem submodule_quotient_relation_respects_smul_by[R: Ring, M: AddCommGroup](s: Submodule[R, M], r: R) {
    quotient_unary_respects(submodule_quotient_relation(s), submodule_smul_by(s, r))
} by {
    submodule_rel_respects_smul_by(s, r)
    submodule_quotient_relation_rel(s)
    quotient_unary_respects_eq_respects_unary_op(submodule_quotient_relation(s), submodule_smul_by(s, r))
}

/// Left scalar multiplication by a fixed scalar on the submodule quotient.
define submodule_quotient_smul_by[R: Ring, M: AddCommGroup](s: Submodule[R, M], r: R) ->
    QuotientOver[M] -> QuotientOver[M] {
    quotient_over_unary_op(submodule_quotient_relation(s), submodule_smul_by(s, r))
}

/// Left scalar multiplication on the submodule quotient agrees with scalar multiplication of representatives.
theorem submodule_quotient_smul_by_projection[R: Ring, M: AddCommGroup](s: Submodule[R, M], r: R, a: M) {
    submodule_quotient_smul_by(s, r,
        quotient_over_mk(submodule_quotient_relation(s), a)
    ) = quotient_over_mk(submodule_quotient_relation(s), s.carrier.smul(r, a))
} by {
    quotient_over_unary_op_projection(submodule_quotient_relation(s), submodule_smul_by(s, r), a)
    submodule_smul_by(s, r, a) = s.carrier.smul(r, a)
}

/// Left scalar multiplication on the submodule quotient is independent of representative.
theorem submodule_quotient_smul_by_projection_compatible[R: Ring, M: AddCommGroup](s: Submodule[R, M], r: R, a: M, b: M) {
    submodule_rel(s, a, b) implies
    submodule_quotient_smul_by(s, r, quotient_over_mk(submodule_quotient_relation(s), a)) =
        submodule_quotient_smul_by(s, r, quotient_over_mk(submodule_quotient_relation(s), b))
} by {
    if submodule_rel(s, a, b) {
        submodule_quotient_smul_by_projection(s, r, a)
        submodule_quotient_smul_by_projection(s, r, b)
        submodule_rel_smul_step(s, r, a, b)
    }
}

/// Addition on the submodule quotient.
define submodule_quotient_add[R: Ring, M: AddCommGroup](s: Submodule[R, M]) ->
    (QuotientOver[M], QuotientOver[M]) -> QuotientOver[M] {
    add_subgroup_quotient_add(submodule_to_add_subgroup(s))
}

/// Negation on the submodule quotient.
define submodule_quotient_neg[R: Ring, M: AddCommGroup](s: Submodule[R, M]) ->
    QuotientOver[M] -> QuotientOver[M] {
    add_subgroup_quotient_neg(submodule_to_add_subgroup(s))
}

/// The zero element of the submodule quotient.
define submodule_quotient_zero[R: Ring, M: AddCommGroup](s: Submodule[R, M]) -> QuotientOver[M] {
    add_subgroup_quotient_zero(submodule_to_add_subgroup(s))
}

/// Addition in the submodule quotient agrees with addition of representatives.
theorem submodule_quotient_add_projection[R: Ring, M: AddCommGroup](s: Submodule[R, M], a: M, b: M) {
    submodule_quotient_add(s,
        quotient_over_mk(submodule_quotient_relation(s), a),
        quotient_over_mk(submodule_quotient_relation(s), b)
    ) = quotient_over_mk(submodule_quotient_relation(s), a + b)
} by {
    submodule_quotient_relation(s) = add_subgroup_quotient_relation(submodule_to_add_subgroup(s))
    add_subgroup_quotient_add_projection(submodule_to_add_subgroup(s), a, b)
}

/// Negation in the submodule quotient agrees with negation of a representative.
theorem submodule_quotient_neg_projection[R: Ring, M: AddCommGroup](s: Submodule[R, M], a: M) {
    submodule_quotient_neg(s,
        quotient_over_mk(submodule_quotient_relation(s), a)
    ) = quotient_over_mk(submodule_quotient_relation(s), -a)
} by {
    submodule_quotient_relation(s) = add_subgroup_quotient_relation(submodule_to_add_subgroup(s))
    add_subgroup_quotient_neg_projection(submodule_to_add_subgroup(s), a)
}

/// Scalar multiplication on the submodule quotient distributes over scalar addition.
theorem submodule_quotient_smul_add_left[R: Ring, M: AddCommGroup](s: Submodule[R, M], r: R, t: R, a: M) {
    submodule_quotient_smul_by(s, r + t,
        quotient_over_mk(submodule_quotient_relation(s), a)) =
    submodule_quotient_add(s,
        submodule_quotient_smul_by(s, r, quotient_over_mk(submodule_quotient_relation(s), a)),
        submodule_quotient_smul_by(s, t, quotient_over_mk(submodule_quotient_relation(s), a)))
} by {
    submodule_quotient_smul_by_projection(s, r + t, a)
    submodule_quotient_smul_by_projection(s, r, a)
    submodule_quotient_smul_by_projection(s, t, a)
    module_smul_add_left(s.carrier, r, t, a)
    s.carrier.smul(r + t, a) = s.carrier.smul(r, a) + s.carrier.smul(t, a)
    quotient_over_mk(submodule_quotient_relation(s), s.carrier.smul(r + t, a)) =
        quotient_over_mk(submodule_quotient_relation(s), s.carrier.smul(r, a) + s.carrier.smul(t, a))
    submodule_quotient_add_projection(s, s.carrier.smul(r, a), s.carrier.smul(t, a))
}

/// Scalar multiplication on the submodule quotient distributes over quotient addition.
theorem submodule_quotient_smul_add_right[R: Ring, M: AddCommGroup](s: Submodule[R, M], r: R, a: M, b: M) {
    submodule_quotient_smul_by(s, r,
        submodule_quotient_add(s,
            quotient_over_mk(submodule_quotient_relation(s), a),
            quotient_over_mk(submodule_quotient_relation(s), b))) =
    submodule_quotient_add(s,
        submodule_quotient_smul_by(s, r, quotient_over_mk(submodule_quotient_relation(s), a)),
        submodule_quotient_smul_by(s, r, quotient_over_mk(submodule_quotient_relation(s), b)))
} by {
    submodule_quotient_add_projection(s, a, b)
    submodule_quotient_smul_by_projection(s, r, a + b)
    submodule_quotient_smul_by_projection(s, r, a)
    submodule_quotient_smul_by_projection(s, r, b)
    module_smul_add_right(s.carrier, r, a, b)
    s.carrier.smul(r, a + b) = s.carrier.smul(r, a) + s.carrier.smul(r, b)
    quotient_over_mk(submodule_quotient_relation(s), s.carrier.smul(r, a + b)) =
        quotient_over_mk(submodule_quotient_relation(s), s.carrier.smul(r, a) + s.carrier.smul(r, b))
    submodule_quotient_add_projection(s, s.carrier.smul(r, a), s.carrier.smul(r, b))
}

/// Scalar multiplication on the submodule quotient is compatible with ring multiplication.
theorem submodule_quotient_smul_assoc[R: Ring, M: AddCommGroup](s: Submodule[R, M], r: R, t: R, a: M) {
    submodule_quotient_smul_by(s, r * t,
        quotient_over_mk(submodule_quotient_relation(s), a)) =
    submodule_quotient_smul_by(s, r,
        submodule_quotient_smul_by(s, t, quotient_over_mk(submodule_quotient_relation(s), a)))
} by {
    submodule_quotient_smul_by_projection(s, r * t, a)
    submodule_quotient_smul_by_projection(s, t, a)
    submodule_quotient_smul_by_projection(s, r, s.carrier.smul(t, a))
    module_smul_assoc(s.carrier, r, t, a)
    s.carrier.smul(r * t, a) = s.carrier.smul(r, s.carrier.smul(t, a))
    quotient_over_mk(submodule_quotient_relation(s), s.carrier.smul(r * t, a)) =
        quotient_over_mk(submodule_quotient_relation(s), s.carrier.smul(r, s.carrier.smul(t, a)))
}

/// The ring identity acts as the identity scalar on the submodule quotient.
theorem submodule_quotient_smul_one[R: Ring, M: AddCommGroup](s: Submodule[R, M], a: M) {
    submodule_quotient_smul_by(s, R.1,
        quotient_over_mk(submodule_quotient_relation(s), a)) =
        quotient_over_mk(submodule_quotient_relation(s), a)
} by {
    submodule_quotient_smul_by_projection(s, R.1, a)
    module_smul_one(s.carrier, a)
    s.carrier.smul(R.1, a) = a
    quotient_over_mk(submodule_quotient_relation(s), s.carrier.smul(R.1, a)) =
        quotient_over_mk(submodule_quotient_relation(s), a)
}

/// The canonical projection from a module to its submodule quotient.
define submodule_quotient_mk[R: Ring, M: AddCommGroup](s: Submodule[R, M]) -> M -> QuotientOver[M] {
    function(a: M) {
        quotient_over_mk(submodule_quotient_relation(s), a)
    }
}

/// The canonical projection unfolds to the quotient maker.
theorem submodule_quotient_mk_def[R: Ring, M: AddCommGroup](s: Submodule[R, M], a: M) {
    submodule_quotient_mk(s)(a) = quotient_over_mk(submodule_quotient_relation(s), a)
}

/// Two elements have equal images under the canonical projection if and only if their
/// difference lies in the submodule.
theorem submodule_quotient_mk_eq_iff_in_submodule[R: Ring, M: AddCommGroup](
    s: Submodule[R, M], a: M, b: M
) {
    (submodule_quotient_mk(s)(a) = submodule_quotient_mk(s)(b)) = s.contains(a - b)
} by {
    submodule_quotient_mk_def(s, a)
    submodule_quotient_mk_def(s, b)
    quotient_over_mk_eq_iff_rel(submodule_quotient_relation(s), a, b)
    submodule_quotient_relation_rel(s)
    submodule_quotient_relation(s).rel(a, b) = submodule_rel(s, a, b)
    submodule_rel(s, a, b) = s.contains(a - b)
}

/// The image of zero under the canonical projection is the quotient zero.
theorem submodule_quotient_mk_zero[R: Ring, M: AddCommGroup](s: Submodule[R, M]) {
    submodule_quotient_mk(s)(M.0) = submodule_quotient_zero(s)
} by {
    submodule_quotient_mk_def(s, M.0)
    submodule_quotient_zero(s) = add_subgroup_quotient_zero(submodule_to_add_subgroup(s))
    add_subgroup_quotient_zero(submodule_to_add_subgroup(s)) =
        quotient_over_mk(add_subgroup_quotient_relation(submodule_to_add_subgroup(s)), M.0)
    submodule_quotient_relation(s) = add_subgroup_quotient_relation(submodule_to_add_subgroup(s))
}

/// An element maps to the quotient zero under the canonical projection if and only if
/// it lies in the submodule. This identifies the kernel of the canonical projection
/// (at the representative level) with the originating submodule.
theorem submodule_quotient_mk_eq_zero_iff_in_submodule[R: Ring, M: AddCommGroup](
    s: Submodule[R, M], a: M
) {
    (submodule_quotient_mk(s)(a) = submodule_quotient_zero(s)) = s.contains(a)
} by {
    submodule_quotient_mk_zero(s)
    submodule_quotient_mk_eq_iff_in_submodule(s, a, M.0)
    a - M.0 = a
}

/// The canonical projection sends a sum to the quotient sum of images.
theorem submodule_quotient_mk_add[R: Ring, M: AddCommGroup](
    s: Submodule[R, M], a: M, b: M
) {
    submodule_quotient_mk(s)(a + b) =
        submodule_quotient_add(s, submodule_quotient_mk(s)(a), submodule_quotient_mk(s)(b))
} by {
    submodule_quotient_mk_def(s, a)
    submodule_quotient_mk_def(s, b)
    submodule_quotient_mk_def(s, a + b)
    submodule_quotient_add_projection(s, a, b)
}

/// The canonical projection sends a negation to the quotient negation of the image.
theorem submodule_quotient_mk_neg[R: Ring, M: AddCommGroup](
    s: Submodule[R, M], a: M
) {
    submodule_quotient_mk(s)(-a) = submodule_quotient_neg(s, submodule_quotient_mk(s)(a))
} by {
    submodule_quotient_mk_def(s, a)
    submodule_quotient_mk_def(s, -a)
    submodule_quotient_neg_projection(s, a)
}

/// The canonical projection intertwines scalar multiplication on the module and the
/// induced scalar action on the quotient.
theorem submodule_quotient_mk_smul[R: Ring, M: AddCommGroup](
    s: Submodule[R, M], r: R, a: M
) {
    submodule_quotient_mk(s)(s.carrier.smul(r, a)) =
        submodule_quotient_smul_by(s, r, submodule_quotient_mk(s)(a))
} by {
    submodule_quotient_mk_def(s, a)
    submodule_quotient_mk_def(s, s.carrier.smul(r, a))
    submodule_quotient_smul_by_projection(s, r, a)
}

/// Two elements of a submodule lie in the same coset if and only if their difference
/// is in the submodule.
theorem submodule_quotient_mk_eq_of_diff_in_submodule[R: Ring, M: AddCommGroup](
    s: Submodule[R, M], a: M, b: M
) {
    s.contains(a - b) implies submodule_quotient_mk(s)(a) = submodule_quotient_mk(s)(b)
} by {
    submodule_quotient_mk_eq_iff_in_submodule(s, a, b)
}

/// If two elements have the same image under the canonical projection, their difference
/// lies in the submodule.
theorem submodule_diff_in_submodule_of_quotient_mk_eq[R: Ring, M: AddCommGroup](
    s: Submodule[R, M], a: M, b: M
) {
    submodule_quotient_mk(s)(a) = submodule_quotient_mk(s)(b) implies s.contains(a - b)
} by {
    submodule_quotient_mk_eq_iff_in_submodule(s, a, b)
}

/// Membership in the preimage of a submodule under a function.
define linear_map_preimage[R: Ring, M: AddCommGroup, N: AddCommGroup](
    t: Submodule[R, N], f: M -> N, x: M
) -> Bool {
    t.contains(f(x))
}

/// The preimage of a submodule under a linear map contains the additive identity.
theorem linear_map_preimage_zero_constraint[R: Ring, M: AddCommGroup, N: AddCommGroup](
    src: Module[R, M], t: Submodule[R, N], f: M -> N
) {
    is_linear_map(src, t.carrier, f) implies
        submodule_zero_constraint(src, linear_map_preimage[R, M, N](t, f))
} by {
    if is_linear_map(src, t.carrier, f) {
        linear_map_zero(src, t.carrier, f)
        f(M.0) = N.0
        submodule_contains_zero(t)
        t.contains(N.0)
        t.contains(f(M.0))
        linear_map_preimage[R, M, N](t, f, M.0)
    }
}

/// The preimage of a submodule under a linear map is closed under addition.
theorem linear_map_preimage_add_constraint[R: Ring, M: AddCommGroup, N: AddCommGroup](
    src: Module[R, M], t: Submodule[R, N], f: M -> N
) {
    is_linear_map(src, t.carrier, f) implies
        submodule_add_constraint(src, linear_map_preimage[R, M, N](t, f))
} by {
    if is_linear_map(src, t.carrier, f) {
        forall(x: M, y: M) {
            if linear_map_preimage[R, M, N](t, f, x) and linear_map_preimage[R, M, N](t, f, y) {
                t.contains(f(x))
                t.contains(f(y))
                submodule_contains_add(t, f(x), f(y))
                t.contains(f(x) + f(y))
                linear_map_add(src, t.carrier, f, x, y)
                f(x + y) = f(x) + f(y)
                t.contains(f(x + y))
                linear_map_preimage[R, M, N](t, f, x + y)
            }
        }
    }
}

/// The preimage of a submodule under a linear map is closed under additive negation.
theorem linear_map_preimage_neg_constraint[R: Ring, M: AddCommGroup, N: AddCommGroup](
    src: Module[R, M], t: Submodule[R, N], f: M -> N
) {
    is_linear_map(src, t.carrier, f) implies
        submodule_neg_constraint(src, linear_map_preimage[R, M, N](t, f))
} by {
    if is_linear_map(src, t.carrier, f) {
        forall(x: M) {
            if linear_map_preimage[R, M, N](t, f, x) {
                t.contains(f(x))
                submodule_contains_neg(t, f(x))
                t.contains(-f(x))
                linear_map_neg(src, t.carrier, f, x)
                f(-x) = -f(x)
                t.contains(f(-x))
                linear_map_preimage[R, M, N](t, f, -x)
            }
        }
    }
}

/// The preimage of a submodule under a linear map is closed under scalar multiplication.
theorem linear_map_preimage_smul_constraint[R: Ring, M: AddCommGroup, N: AddCommGroup](
    src: Module[R, M], t: Submodule[R, N], f: M -> N
) {
    is_linear_map(src, t.carrier, f) implies
        submodule_smul_constraint(src, linear_map_preimage[R, M, N](t, f))
} by {
    if is_linear_map(src, t.carrier, f) {
        forall(r: R, x: M) {
            if linear_map_preimage[R, M, N](t, f, x) {
                t.contains(f(x))
                submodule_contains_smul(t, r, f(x))
                t.carrier.smul(r, f(x)) = t.carrier.smul(r, f(x))
                t.contains(t.carrier.smul(r, f(x)))
                linear_map_smul(src, t.carrier, f, r, x)
                f(src.smul(r, x)) = t.carrier.smul(r, f(x))
                t.contains(f(src.smul(r, x)))
                linear_map_preimage[R, M, N](t, f, src.smul(r, x))
            }
        }
    }
}

/// Membership in the preimage of the zero submodule under a function is membership in the kernel.
theorem linear_map_preimage_zero_submodule_eq_kernel[R: Ring, M: AddCommGroup, N: AddCommGroup](
    dst: Module[R, N], f: M -> N, x: M
) {
    linear_map_preimage[R, M, N](zero_submodule(dst), f, x) = linear_map_kernel[R, M, N](f, x)
} by {
    linear_map_preimage[R, M, N](zero_submodule(dst), f, x) = zero_submodule(dst).contains(f(x))
    zero_submodule_contains_eq(dst, f(x))
    zero_submodule(dst).contains(f(x)) = (f(x) = N.0)
    linear_map_kernel[R, M, N](f, x) = (f(x) = N.0)
}

/// Membership in the preimage of the full submodule under any function is universal.
theorem linear_map_preimage_full_submodule[R: Ring, M: AddCommGroup, N: AddCommGroup](
    dst: Module[R, N], f: M -> N, x: M
) {
    linear_map_preimage[R, M, N](full_submodule(dst), f, x)
} by {
    full_submodule_contains_everything(dst, f(x))
    full_submodule(dst).contains(f(x))
    linear_map_preimage[R, M, N](full_submodule(dst), f, x)
}

/// The preimage predicate when the map is linear, and the full predicate otherwise.
define linear_map_preimage_submodule_contains[R: Ring, M: AddCommGroup, N: AddCommGroup](
    src: Module[R, M], t: Submodule[R, N], f: M -> N, x: M
) -> Bool {
    if is_linear_map(src, t.carrier, f) {
        linear_map_preimage[R, M, N](t, f, x)
    } else {
        full_submodule_contains[M](x)
    }
}

/// The preimage-submodule predicate agrees with the preimage predicate for a linear map.
theorem linear_map_preimage_submodule_contains_eq_of_linear[R: Ring, M: AddCommGroup, N: AddCommGroup](
    src: Module[R, M], t: Submodule[R, N], f: M -> N, x: M
) {
    is_linear_map(src, t.carrier, f) implies
        linear_map_preimage_submodule_contains(src, t, f, x) = linear_map_preimage[R, M, N](t, f, x)
} by {
    if is_linear_map(src, t.carrier, f) {
        linear_map_preimage_submodule_contains(src, t, f, x) = linear_map_preimage[R, M, N](t, f, x)
    }
}

/// The preimage-submodule predicate is universal when the map is not known to be linear.
theorem linear_map_preimage_submodule_contains_eq_full_of_not_linear[R: Ring, M: AddCommGroup, N: AddCommGroup](
    src: Module[R, M], t: Submodule[R, N], f: M -> N, x: M
) {
    not(is_linear_map(src, t.carrier, f)) implies
        linear_map_preimage_submodule_contains(src, t, f, x) = full_submodule_contains[M](x)
} by {
    if not(is_linear_map(src, t.carrier, f)) {
        linear_map_preimage_submodule_contains(src, t, f, x) = full_submodule_contains[M](x)
    }
}

/// The total preimage-submodule predicate is a submodule predicate.
theorem linear_map_preimage_submodule_is_submodule[R: Ring, M: AddCommGroup, N: AddCommGroup](
    src: Module[R, M], t: Submodule[R, N], f: M -> N
) {
    is_submodule(src, linear_map_preimage_submodule_contains(src, t, f))
} by {
    let p = linear_map_preimage_submodule_contains(src, t, f)
    is_submodule(src, p) =
        (submodule_zero_constraint(src, p)
         and submodule_add_constraint(src, p)
         and submodule_neg_constraint(src, p)
         and submodule_smul_constraint(src, p))
    if is_linear_map(src, t.carrier, f) {
        linear_map_preimage_zero_constraint(src, t, f)
        submodule_zero_constraint(src, linear_map_preimage[R, M, N](t, f))
        linear_map_preimage_submodule_contains_eq_of_linear(src, t, f, M.0)
        linear_map_preimage_submodule_contains(src, t, f, M.0)
        submodule_zero_constraint(src, linear_map_preimage_submodule_contains(src, t, f))

        linear_map_preimage_add_constraint(src, t, f)
        submodule_add_constraint(src, linear_map_preimage[R, M, N](t, f)) = forall(a: M, b: M) {
            linear_map_preimage[R, M, N](t, f, a) and linear_map_preimage[R, M, N](t, f, b)
                implies linear_map_preimage[R, M, N](t, f, a + b)
        }
        forall(x: M, y: M) {
            if linear_map_preimage_submodule_contains(src, t, f, x)
                and linear_map_preimage_submodule_contains(src, t, f, y) {
                linear_map_preimage_submodule_contains_eq_of_linear(src, t, f, x)
                linear_map_preimage_submodule_contains_eq_of_linear(src, t, f, y)
                linear_map_preimage[R, M, N](t, f, x)
                linear_map_preimage[R, M, N](t, f, y)
                linear_map_preimage[R, M, N](t, f, x + y)
                linear_map_preimage_submodule_contains_eq_of_linear(src, t, f, x + y)
                linear_map_preimage_submodule_contains(src, t, f, x + y)
            }
        }
        submodule_add_constraint(src, linear_map_preimage_submodule_contains(src, t, f))

        linear_map_preimage_neg_constraint(src, t, f)
        submodule_neg_constraint(src, linear_map_preimage[R, M, N](t, f)) = forall(a: M) {
            linear_map_preimage[R, M, N](t, f, a) implies linear_map_preimage[R, M, N](t, f, -a)
        }
        forall(x: M) {
            if linear_map_preimage_submodule_contains(src, t, f, x) {
                linear_map_preimage_submodule_contains_eq_of_linear(src, t, f, x)
                linear_map_preimage[R, M, N](t, f, x)
                linear_map_preimage[R, M, N](t, f, -x)
                linear_map_preimage_submodule_contains_eq_of_linear(src, t, f, -x)
                linear_map_preimage_submodule_contains(src, t, f, -x)
            }
        }
        submodule_neg_constraint(src, linear_map_preimage_submodule_contains(src, t, f))

        linear_map_preimage_smul_constraint(src, t, f)
        submodule_smul_constraint(src, linear_map_preimage[R, M, N](t, f)) = forall(q: R, a: M) {
            linear_map_preimage[R, M, N](t, f, a) implies linear_map_preimage[R, M, N](t, f, src.smul(q, a))
        }
        forall(r: R, x: M) {
            if linear_map_preimage_submodule_contains(src, t, f, x) {
                linear_map_preimage_submodule_contains_eq_of_linear(src, t, f, x)
                linear_map_preimage[R, M, N](t, f, x)
                linear_map_preimage[R, M, N](t, f, src.smul(r, x))
                linear_map_preimage_submodule_contains_eq_of_linear(src, t, f, src.smul(r, x))
                linear_map_preimage_submodule_contains(src, t, f, src.smul(r, x))
            }
        }
        submodule_smul_constraint(src, linear_map_preimage_submodule_contains(src, t, f))
        submodule_zero_constraint(src, p) and submodule_add_constraint(src, p) and submodule_neg_constraint(src, p) and submodule_smul_constraint(src, p)
        is_submodule(src, p)
    } else {
        full_submodule_is_submodule(src)
        forall(x: M) {
            linear_map_preimage_submodule_contains_eq_full_of_not_linear(src, t, f, x)
            linear_map_preimage_submodule_contains(src, t, f, x)
        }
        submodule_zero_constraint(src, linear_map_preimage_submodule_contains(src, t, f))
        forall(x: M, y: M) {
            if linear_map_preimage_submodule_contains(src, t, f, x)
                and linear_map_preimage_submodule_contains(src, t, f, y) {
                linear_map_preimage_submodule_contains(src, t, f, x + y)
            }
        }
        submodule_add_constraint(src, linear_map_preimage_submodule_contains(src, t, f))
        forall(x: M) {
            if linear_map_preimage_submodule_contains(src, t, f, x) {
                linear_map_preimage_submodule_contains(src, t, f, -x)
            }
        }
        submodule_neg_constraint(src, linear_map_preimage_submodule_contains(src, t, f))
        forall(r: R, x: M) {
            if linear_map_preimage_submodule_contains(src, t, f, x) {
                linear_map_preimage_submodule_contains(src, t, f, src.smul(r, x))
            }
        }
        submodule_smul_constraint(src, linear_map_preimage_submodule_contains(src, t, f))
        is_submodule(src, linear_map_preimage_submodule_contains(src, t, f))
    }
}

/// The preimage submodule of a submodule under a linear map, using the full submodule when the map is not linear.
let linear_map_preimage_submodule[R: Ring, M: AddCommGroup, N: AddCommGroup](src: Module[R, M], t: Submodule[R, N], f: M -> N) -> result: Submodule[R, M] satisfy {
    Submodule.new(src, linear_map_preimage_submodule_contains(src, t, f)) = Option.some(result)
} by {
    linear_map_preimage_submodule_is_submodule(src, t, f)
}

/// The preimage submodule has the source module as its ambient module.
theorem linear_map_preimage_submodule_carrier_eq[R: Ring, M: AddCommGroup, N: AddCommGroup](
    src: Module[R, M], t: Submodule[R, N], f: M -> N
) {
    linear_map_preimage_submodule(src, t, f).carrier = src
}

/// Membership in the preimage submodule is membership of the image in the target submodule for a linear map.
theorem linear_map_preimage_submodule_contains_eq[R: Ring, M: AddCommGroup, N: AddCommGroup](
    src: Module[R, M], t: Submodule[R, N], f: M -> N, x: M
) {
    is_linear_map(src, t.carrier, f) implies
        linear_map_preimage_submodule(src, t, f).contains(x) = t.contains(f(x))
} by {
    if is_linear_map(src, t.carrier, f) {
        let p = linear_map_preimage_submodule_contains(src, t, f)
        linear_map_preimage_submodule_is_submodule(src, t, f)
        is_submodule(src, p)
        let s: Submodule[R, M] satisfy {
            Submodule.new(src, p) = Option.some(s)
        }
        linear_map_preimage_submodule(src, t, f) = s
        s.contains(x) = p(x)
        p(x) = linear_map_preimage_submodule_contains(src, t, f, x)
        linear_map_preimage_submodule(src, t, f).contains(x) = linear_map_preimage_submodule_contains(src, t, f, x)
        linear_map_preimage_submodule_contains_eq_of_linear(src, t, f, x)
        linear_map_preimage_submodule_contains(src, t, f, x) = linear_map_preimage[R, M, N](t, f, x)
        linear_map_preimage[R, M, N](t, f, x) = t.contains(f(x))
        linear_map_preimage_submodule(src, t, f).contains(x) = t.contains(f(x))
    }
}

/// An element whose image lies in the target submodule belongs to the preimage submodule.
theorem linear_map_preimage_submodule_contains_of_image_in[R: Ring, M: AddCommGroup, N: AddCommGroup](
    src: Module[R, M], t: Submodule[R, N], f: M -> N, x: M
) {
    is_linear_map(src, t.carrier, f) and t.contains(f(x))
    implies linear_map_preimage_submodule(src, t, f).contains(x)
} by {
    if is_linear_map(src, t.carrier, f) and t.contains(f(x)) {
        linear_map_preimage_submodule_contains_eq(src, t, f, x)
    }
}

/// A member of the preimage submodule has its image in the target submodule.
theorem linear_map_preimage_submodule_image_in_of_contains[R: Ring, M: AddCommGroup, N: AddCommGroup](
    src: Module[R, M], t: Submodule[R, N], f: M -> N, x: M
) {
    is_linear_map(src, t.carrier, f) and linear_map_preimage_submodule(src, t, f).contains(x)
    implies t.contains(f(x))
} by {
    if is_linear_map(src, t.carrier, f) and linear_map_preimage_submodule(src, t, f).contains(x) {
        linear_map_preimage_submodule_contains_eq(src, t, f, x)
    }
}

/// Pointwise membership of the preimage of the full submodule matches the full source submodule.
theorem linear_map_preimage_submodule_full_contains_at[R: Ring, M: AddCommGroup, N: AddCommGroup](
    src: Module[R, M], dst: Module[R, N], f: M -> N, x: M
) {
    is_linear_map(src, dst, f) implies
        linear_map_preimage_submodule(src, full_submodule(dst), f).contains(x) = full_submodule(src).contains(x)
} by {
    if is_linear_map(src, dst, f) {
        full_submodule_carrier_eq(dst)
        full_submodule(dst).carrier = dst
        is_linear_map(src, full_submodule(dst).carrier, f)
        full_submodule_contains_everything(dst, f(x))
        linear_map_preimage_submodule_contains_of_image_in(src, full_submodule(dst), f, x)
        full_submodule_contains_everything(src, x)
    }
}

/// The preimage submodule is monotone in the target submodule.
theorem linear_map_preimage_submodule_mono[R: Ring, M: AddCommGroup, N: AddCommGroup](
    src: Module[R, M], t1: Submodule[R, N], t2: Submodule[R, N], f: M -> N
) {
    t1.carrier = t2.carrier and is_linear_map(src, t1.carrier, f) and submodule_subset(t1, t2)
    implies submodule_subset(
        linear_map_preimage_submodule(src, t1, f),
        linear_map_preimage_submodule(src, t2, f))
} by {
    if t1.carrier = t2.carrier and is_linear_map(src, t1.carrier, f) and submodule_subset(t1, t2) {
        is_linear_map(src, t2.carrier, f)
        forall(x: M) {
            if linear_map_preimage_submodule(src, t1, f).contains(x) {
                linear_map_preimage_submodule_image_in_of_contains(src, t1, f, x)
                t1.contains(f(x))
                submodule_subset(t1, t2) = forall(y: N) {
                    t1.contains(y) implies t2.contains(y)
                }
                t2.contains(f(x))
                linear_map_preimage_submodule_contains_of_image_in(src, t2, f, x)
                linear_map_preimage_submodule(src, t2, f).contains(x)
            }
        }
    }
}

/// Pointwise, the preimage of the zero submodule under a linear map agrees with the kernel submodule.
theorem linear_map_preimage_submodule_zero_eq_kernel_at[R: Ring, M: AddCommGroup, N: AddCommGroup](
    src: Module[R, M], dst: Module[R, N], f: M -> N, x: M
) {
    is_linear_map(src, dst, f) implies linear_map_preimage_submodule(src, zero_submodule(dst), f).contains(x) = linear_map_kernel_submodule(src, dst, f).contains(x)
} by {
    if is_linear_map(src, dst, f) {
        zero_submodule_carrier_eq(dst)
        zero_submodule(dst).carrier = dst
        is_linear_map(src, zero_submodule(dst).carrier, f)
        linear_map_preimage_submodule_contains_eq(src, zero_submodule(dst), f, x)
        linear_map_preimage_submodule(src, zero_submodule(dst), f).contains(x) = zero_submodule(dst).contains(f(x))
        zero_submodule_contains_eq(dst, f(x))
        zero_submodule(dst).contains(f(x)) = (f(x) = N.0)
        linear_map_kernel_submodule_contains_eq(src, dst, f, x)
        linear_map_kernel_submodule(src, dst, f).contains(x) = (f(x) = N.0)
        linear_map_preimage_submodule(src, zero_submodule(dst), f).contains(x) = (f(x) = N.0)
        linear_map_preimage_submodule(src, zero_submodule(dst), f).contains(x) = linear_map_kernel_submodule(src, dst, f).contains(x)
    }
}

/// Under a linear map, the preimage of the zero submodule is contained in the kernel submodule.
theorem linear_map_preimage_submodule_zero_subset_kernel[R: Ring, M: AddCommGroup, N: AddCommGroup](
    src: Module[R, M], dst: Module[R, N], f: M -> N
) {
    is_linear_map(src, dst, f) implies
        submodule_subset(
            linear_map_preimage_submodule(src, zero_submodule(dst), f),
            linear_map_kernel_submodule(src, dst, f))
} by {
    if is_linear_map(src, dst, f) {
        forall(x: M) {
            if linear_map_preimage_submodule(src, zero_submodule(dst), f).contains(x) {
                linear_map_preimage_submodule_zero_eq_kernel_at(src, dst, f, x)
                linear_map_kernel_submodule(src, dst, f).contains(x)
            }
        }
    }
}

/// Under a linear map, the kernel submodule is contained in the preimage of the zero submodule.
theorem linear_map_kernel_subset_preimage_submodule_zero[R: Ring, M: AddCommGroup, N: AddCommGroup](
    src: Module[R, M], dst: Module[R, N], f: M -> N
) {
    is_linear_map(src, dst, f) implies
        submodule_subset(
            linear_map_kernel_submodule(src, dst, f),
            linear_map_preimage_submodule(src, zero_submodule(dst), f))
} by {
    if is_linear_map(src, dst, f) {
        forall(x: M) {
            if linear_map_kernel_submodule(src, dst, f).contains(x) {
                linear_map_preimage_submodule_zero_eq_kernel_at(src, dst, f, x)
                linear_map_preimage_submodule(src, zero_submodule(dst), f).contains(x)
            }
        }
    }
}

/// Under a linear map, the preimage of the full submodule is contained in the full source submodule.
theorem linear_map_preimage_submodule_full_subset[R: Ring, M: AddCommGroup, N: AddCommGroup](
    src: Module[R, M], dst: Module[R, N], f: M -> N
) {
    is_linear_map(src, dst, f) implies
        submodule_subset(
            linear_map_preimage_submodule(src, full_submodule(dst), f),
            full_submodule(src))
} by {
    if is_linear_map(src, dst, f) {
        forall(x: M) {
            if linear_map_preimage_submodule(src, full_submodule(dst), f).contains(x) {
                full_submodule_contains_everything(src, x)
            }
        }
    }
}

/// Under a linear map, the full source submodule is contained in the preimage of the full submodule.
theorem full_submodule_subset_linear_map_preimage_full[R: Ring, M: AddCommGroup, N: AddCommGroup](
    src: Module[R, M], dst: Module[R, N], f: M -> N
) {
    is_linear_map(src, dst, f) implies
        submodule_subset(
            full_submodule(src),
            linear_map_preimage_submodule(src, full_submodule(dst), f))
} by {
    if is_linear_map(src, dst, f) {
        forall(x: M) {
            if full_submodule(src).contains(x) {
                linear_map_preimage_submodule_full_contains_at(src, dst, f, x)
                linear_map_preimage_submodule(src, full_submodule(dst), f).contains(x)
            }
        }
    }
}
/// Under a linear equivalence, membership in the kernel is equivalent to being zero.
theorem linear_equiv_kernel_iff_zero[R: Ring, M: AddCommGroup, N: AddCommGroup](
    src: Module[R, M], dst: Module[R, N], f: M -> N, x: M
) {
    is_linear_equiv(src, dst, f) implies (linear_map_kernel[R, M, N](f, x) iff x = M.0)
} by {
    if is_linear_equiv(src, dst, f) {
        linear_equiv_apply_eq_zero_iff_eq_zero(src, dst, f, x)
        if linear_map_kernel[R, M, N](f, x) {
            f(x) = N.0
            x = M.0
        }
        if x = M.0 {
            f(x) = N.0
            linear_map_kernel[R, M, N](f, x)
        }
        linear_map_kernel[R, M, N](f, x) iff x = M.0
    }
}

/// Under a linear equivalence, every destination point lies in the image.
theorem linear_equiv_image_full_at[R: Ring, M: AddCommGroup, N: AddCommGroup](
    src: Module[R, M], dst: Module[R, N], f: M -> N, y: N
) {
    is_linear_equiv(src, dst, f) implies linear_map_image[M, N](f, y)
} by {
    if is_linear_equiv(src, dst, f) {
        linear_equiv_is_surjective(src, dst, f)
        is_surjective_fn(f) = forall(b: N) {
            exists(a: M) {
                f(a) = b
            }
        }
        exists(a: M) {
            f(a) = y
        }
        linear_map_image[M, N](f, y)
    }
}

/// Under a linear equivalence, the kernel submodule contains an element iff that element is zero.
theorem linear_equiv_kernel_submodule_contains_iff_zero[R: Ring, M: AddCommGroup, N: AddCommGroup](
    src: Module[R, M], dst: Module[R, N], f: M -> N, x: M
) {
    is_linear_equiv(src, dst, f) implies
        (linear_map_kernel_submodule(src, dst, f).contains(x) iff x = M.0)
} by {
    if is_linear_equiv(src, dst, f) {
        linear_equiv_is_linear_map(src, dst, f)
        linear_map_kernel_submodule_contains_eq(src, dst, f, x)
        linear_map_kernel_submodule(src, dst, f).contains(x) = (f(x) = N.0)
        linear_equiv_apply_eq_zero_iff_eq_zero(src, dst, f, x)
        if linear_map_kernel_submodule(src, dst, f).contains(x) {
            f(x) = N.0
            x = M.0
        }
        if x = M.0 {
            f(x) = N.0
            linear_map_kernel_submodule(src, dst, f).contains(x)
        }
        linear_map_kernel_submodule(src, dst, f).contains(x) iff x = M.0
    }
}

/// Under a linear equivalence, the image submodule contains every destination element.
theorem linear_equiv_image_submodule_contains[R: Ring, M: AddCommGroup, N: AddCommGroup](
    src: Module[R, M], dst: Module[R, N], f: M -> N, y: N
) {
    is_linear_equiv(src, dst, f) implies
        linear_map_image_submodule(src, dst, f).contains(y)
} by {
    if is_linear_equiv(src, dst, f) {
        linear_equiv_is_linear_map(src, dst, f)
        linear_map_image_submodule_contains_eq(src, dst, f, y)
        linear_equiv_image_full_at(src, dst, f, y)
        linear_map_image_submodule(src, dst, f).contains(y) = linear_map_image[M, N](f, y)
        linear_map_image_submodule(src, dst, f).contains(y)
    }
}

/// Under a linear equivalence, the kernel submodule is contained in the zero submodule of the source.
theorem linear_equiv_kernel_submodule_subset_zero_submodule[R: Ring, M: AddCommGroup, N: AddCommGroup](
    src: Module[R, M], dst: Module[R, N], f: M -> N
) {
    is_linear_equiv(src, dst, f) implies
        submodule_subset(linear_map_kernel_submodule(src, dst, f), zero_submodule(src))
} by {
    if is_linear_equiv(src, dst, f) {
        forall(x: M) {
            if linear_map_kernel_submodule(src, dst, f).contains(x) {
                linear_equiv_kernel_submodule_contains_iff_zero(src, dst, f, x)
                x = M.0
                zero_submodule_contains_eq(src, x)
                zero_submodule(src).contains(x)
            }
        }
    }
}

/// Under a linear equivalence, the zero submodule of the source is contained in the kernel submodule.
theorem zero_submodule_subset_linear_equiv_kernel_submodule[R: Ring, M: AddCommGroup, N: AddCommGroup](
    src: Module[R, M], dst: Module[R, N], f: M -> N
) {
    is_linear_equiv(src, dst, f) implies
        submodule_subset(zero_submodule(src), linear_map_kernel_submodule(src, dst, f))
} by {
    if is_linear_equiv(src, dst, f) {
        forall(x: M) {
            if zero_submodule(src).contains(x) {
                zero_submodule_contains_eq(src, x)
                x = M.0
                linear_equiv_kernel_submodule_contains_iff_zero(src, dst, f, x)
                linear_map_kernel_submodule(src, dst, f).contains(x)
            }
        }
    }
}

/// Under a linear equivalence, the full submodule of the destination is contained in the image submodule.
theorem full_submodule_subset_linear_equiv_image_submodule[R: Ring, M: AddCommGroup, N: AddCommGroup](
    src: Module[R, M], dst: Module[R, N], f: M -> N
) {
    is_linear_equiv(src, dst, f) implies
        submodule_subset(full_submodule(dst), linear_map_image_submodule(src, dst, f))
} by {
    if is_linear_equiv(src, dst, f) {
        forall(y: N) {
            if full_submodule(dst).contains(y) {
                linear_equiv_image_submodule_contains(src, dst, f, y)
                linear_map_image_submodule(src, dst, f).contains(y)
            }
        }
    }
}

/// The kernel of a bundled module homomorphism, as a submodule of the source.
define module_hom_kernel[R: Ring, M: AddCommGroup, N: AddCommGroup](
    f: ModuleHom[R, M, N]
) -> Submodule[R, M] {
    linear_map_kernel_submodule(f.src, f.dst, f.hom)
}

/// The kernel submodule of a bundled module homomorphism has the source module as its ambient module.
theorem module_hom_kernel_carrier[R: Ring, M: AddCommGroup, N: AddCommGroup](
    f: ModuleHom[R, M, N]
) {
    module_hom_kernel(f).carrier = f.src
} by {
    linear_map_kernel_submodule_carrier_eq(f.src, f.dst, f.hom)
}

/// Membership in the kernel of a bundled module homomorphism is equality of the image to zero.
theorem module_hom_kernel_contains_eq[R: Ring, M: AddCommGroup, N: AddCommGroup](
    f: ModuleHom[R, M, N], x: M
) {
    module_hom_kernel(f).contains(x) = (f.hom(x) = N.0)
} by {
    module_hom_is_linear_map(f)
    linear_map_kernel_submodule_contains_eq(f.src, f.dst, f.hom, x)
}

/// An element whose image is zero belongs to the kernel of a bundled module homomorphism.
theorem module_hom_kernel_contains_of_map_eq_zero[R: Ring, M: AddCommGroup, N: AddCommGroup](
    f: ModuleHom[R, M, N], x: M
) {
    f.hom(x) = N.0 implies module_hom_kernel(f).contains(x)
} by {
    if f.hom(x) = N.0 {
        module_hom_kernel_contains_eq(f, x)
    }
}

/// A member of the kernel of a bundled module homomorphism has image zero.
theorem module_hom_kernel_map_eq_zero[R: Ring, M: AddCommGroup, N: AddCommGroup](
    f: ModuleHom[R, M, N], x: M
) {
    module_hom_kernel(f).contains(x) implies f.hom(x) = N.0
} by {
    if module_hom_kernel(f).contains(x) {
        module_hom_kernel_contains_eq(f, x)
    }
}

/// The image of a bundled module homomorphism, as a submodule of the destination.
define module_hom_image[R: Ring, M: AddCommGroup, N: AddCommGroup](
    f: ModuleHom[R, M, N]
) -> Submodule[R, N] {
    linear_map_image_submodule(f.src, f.dst, f.hom)
}

/// The image submodule of a bundled module homomorphism has the destination module as its ambient module.
theorem module_hom_image_carrier[R: Ring, M: AddCommGroup, N: AddCommGroup](
    f: ModuleHom[R, M, N]
) {
    module_hom_image(f).carrier = f.dst
} by {
    linear_map_image_submodule_carrier_eq(f.src, f.dst, f.hom)
}

/// Every image value of a bundled module homomorphism lies in its image submodule.
theorem module_hom_image_contains_value[R: Ring, M: AddCommGroup, N: AddCommGroup](
    f: ModuleHom[R, M, N], x: M
) {
    module_hom_image(f).contains(f.hom(x))
} by {
    module_hom_is_linear_map(f)
    linear_map_image_submodule_contains_apply(f.src, f.dst, f.hom, x)
}

/// Membership in the image submodule of a bundled module homomorphism is membership in the image predicate.
theorem module_hom_image_contains_eq[R: Ring, M: AddCommGroup, N: AddCommGroup](
    f: ModuleHom[R, M, N], y: N
) {
    module_hom_image(f).contains(y) = linear_map_image[M, N](f.hom, y)
} by {
    module_hom_is_linear_map(f)
    linear_map_image_submodule_contains_eq(f.src, f.dst, f.hom, y)
}

/// An element of the image submodule of a bundled module homomorphism is the image of some source point.
theorem module_hom_image_elim[R: Ring, M: AddCommGroup, N: AddCommGroup](
    f: ModuleHom[R, M, N], y: N
) {
    module_hom_image(f).contains(y) implies exists(x: M) { f.hom(x) = y }
} by {
    if module_hom_image(f).contains(y) {
        module_hom_is_linear_map(f)
        linear_map_image_of_submodule_contains(f.src, f.dst, f.hom, y)
    }
}

/// The image submodule of a bundled module homomorphism is contained in a submodule precisely when every map value belongs to it.
theorem module_hom_image_subset_iff[R: Ring, M: AddCommGroup, N: AddCommGroup](
    f: ModuleHom[R, M, N], s: Submodule[R, N]
) {
    submodule_subset(module_hom_image(f), s) = forall(x: M) { s.contains(f.hom(x)) }
} by {
    module_hom_is_linear_map(f)
    linear_map_image_submodule_subset_iff(f.src, f.dst, f.hom, s)
}

/// The kernel submodule of a bundled module homomorphism is contained in a source submodule precisely when every source element of the kernel belongs to it; here we record the simpler kernel-zero characterisation: an injective bundled homomorphism has trivial kernel pointwise.
theorem module_hom_kernel_contains_iff_map_eq_zero[R: Ring, M: AddCommGroup, N: AddCommGroup](
    f: ModuleHom[R, M, N], x: M
) {
    module_hom_kernel(f).contains(x) iff f.hom(x) = N.0
} by {
    module_hom_kernel_contains_eq(f, x)
}

/// An injective bundled module homomorphism has trivial kernel: kernel membership implies the source element is zero.
theorem module_hom_kernel_implies_zero_of_injective[R: Ring, M: AddCommGroup, N: AddCommGroup](
    f: ModuleHom[R, M, N], x: M
) {
    is_injective_fn(f.hom) and module_hom_kernel(f).contains(x) implies x = M.0
} by {
    if is_injective_fn(f.hom) and module_hom_kernel(f).contains(x) {
        module_hom_kernel_map_eq_zero(f, x)
        f.hom(x) = N.0
        module_hom_zero(f)
        f.hom(M.0) = N.0
        f.hom(x) = f.hom(M.0)
        is_injective_fn(f.hom) = forall(a: M, b: M) {
            f.hom(a) = f.hom(b) implies a = b
        }
        x = M.0
    }
}

/// An injective bundled module homomorphism has its kernel submodule contained in the zero submodule.
theorem module_hom_kernel_subset_zero_of_injective[R: Ring, M: AddCommGroup, N: AddCommGroup](
    f: ModuleHom[R, M, N]
) {
    is_injective_fn(f.hom) implies
        submodule_subset(module_hom_kernel(f), zero_submodule(f.src))
} by {
    if is_injective_fn(f.hom) {
        forall(x: M) {
            if module_hom_kernel(f).contains(x) {
                module_hom_kernel_implies_zero_of_injective(f, x)
                x = M.0
                zero_submodule_contains_eq(f.src, x)
                zero_submodule(f.src).contains(x)
            }
        }
    }
}

/// A bundled module homomorphism whose kernel submodule equals the zero submodule is injective on its underlying function.
theorem module_hom_injective_of_kernel_eq_zero[R: Ring, M: AddCommGroup, N: AddCommGroup](
    f: ModuleHom[R, M, N]
) {
    module_hom_kernel(f) = zero_submodule(f.src) implies is_injective_fn(f.hom)
} by {
    if module_hom_kernel(f) = zero_submodule(f.src) {
        module_hom_is_linear_map(f)
        forall(x: M, y: M) {
            if f.hom(x) = f.hom(y) {
                module_hom_sub(f, x, y)
                f.hom(x - y) = f.hom(x) - f.hom(y)
                f.hom(x) - f.hom(y) = N.0
                f.hom(x - y) = N.0
                module_hom_kernel_contains_of_map_eq_zero(f, x - y)
                module_hom_kernel(f).contains(x - y)
                zero_submodule(f.src).contains(x - y)
                zero_submodule_contains_eq(f.src, x - y)
                x - y = M.0
                x - y + y = M.0 + y
                x + (-y + y) = y
                -y + y = M.0
                x + M.0 = y
                x = y
            }
        }
    }
}

/// The full submodule of the destination is contained in the image submodule of a surjective bundled module homomorphism.
theorem full_submodule_subset_module_hom_image_of_surjective[R: Ring, M: AddCommGroup, N: AddCommGroup](
    f: ModuleHom[R, M, N]
) {
    is_surjective_fn(f.hom) implies
        submodule_subset(full_submodule(f.dst), module_hom_image(f))
} by {
    if is_surjective_fn(f.hom) {
        module_hom_is_linear_map(f)
        full_submodule_subset_linear_map_image_submodule_of_surjective(f.src, f.dst, f.hom)
    }
}

/// If the image submodule of a bundled module homomorphism equals the full submodule, then every destination point lies in the image predicate.
theorem module_hom_image_of_image_eq_full[R: Ring, M: AddCommGroup, N: AddCommGroup](
    f: ModuleHom[R, M, N], y: N
) {
    module_hom_image(f) = full_submodule(f.dst)
        implies exists(x: M) { f.hom(x) = y }
} by {
    if module_hom_image(f) = full_submodule(f.dst) {
        module_hom_is_linear_map(f)
        linear_map_image_of_image_submodule_eq_full(f.src, f.dst, f.hom, y)
        linear_map_image[M, N](f.hom, y)
    }
}

/// The kernel of the identity module homomorphism contains a source point precisely when the point is zero.
theorem module_hom_identity_kernel_contains_eq[R: Ring, M: AddCommGroup](
    m: Module[R, M], x: M
) {
    module_hom_kernel(module_hom_identity(m)).contains(x) = (x = M.0)
} by {
    module_hom_kernel_contains_eq(module_hom_identity(m), x)
    module_hom_identity_hom_at(m, x)
    module_hom_kernel(module_hom_identity(m)).contains(x) = (module_hom_identity(m).hom(x) = M.0)
    module_hom_identity(m).hom(x) = x
}

/// The kernel of the identity module homomorphism and the zero submodule share membership at every point.
theorem module_hom_identity_kernel_contains_eq_zero_submodule[R: Ring, M: AddCommGroup](
    m: Module[R, M], x: M
) {
    module_hom_kernel(module_hom_identity(m)).contains(x) = zero_submodule(m).contains(x)
} by {
    module_hom_identity_kernel_contains_eq(m, x)
    zero_submodule_contains_eq(m, x)
}

/// Every destination point lies in the image of the identity module homomorphism.
theorem module_hom_identity_image_contains[R: Ring, M: AddCommGroup](
    m: Module[R, M], x: M
) {
    module_hom_image(module_hom_identity(m)).contains(x)
} by {
    module_hom_image_contains_value(module_hom_identity(m), x)
    module_hom_identity_hom_at(m, x)
    module_hom_identity(m).hom(x) = x
}

/// The image of the identity module homomorphism and the full submodule share membership at every point.
theorem module_hom_identity_image_contains_eq_full_submodule[R: Ring, M: AddCommGroup](
    m: Module[R, M], x: M
) {
    module_hom_image(module_hom_identity(m)).contains(x) = full_submodule(m).contains(x)
} by {
    module_hom_identity_image_contains(m, x)
    full_submodule_contains_eq(m, x)
}

/// Every source point lies in the kernel of the trivial module homomorphism.
theorem module_hom_trivial_kernel_contains[R: Ring, M: AddCommGroup, N: AddCommGroup](
    src: Module[R, M], dst: Module[R, N], x: M
) {
    module_hom_kernel(module_hom_trivial(src, dst)).contains(x)
} by {
    module_hom_kernel_contains_eq(module_hom_trivial(src, dst), x)
    module_hom_trivial_hom_at(src, dst, x)
    module_hom_trivial(src, dst).hom(x) = N.0
}

/// The kernel of the trivial module homomorphism and the full source submodule share membership at every point.
theorem module_hom_trivial_kernel_contains_eq_full_submodule[R: Ring, M: AddCommGroup, N: AddCommGroup](
    src: Module[R, M], dst: Module[R, N], x: M
) {
    module_hom_kernel(module_hom_trivial(src, dst)).contains(x) = full_submodule(src).contains(x)
} by {
    module_hom_trivial_kernel_contains(src, dst, x)
    full_submodule_contains_eq(src, x)
}

/// The kernel of a bundled module homomorphism contains the zero of the source module.
theorem module_hom_kernel_contains_zero[R: Ring, M: AddCommGroup, N: AddCommGroup](
    f: ModuleHom[R, M, N]
) {
    module_hom_kernel(f).contains(M.0)
} by {
    submodule_contains_zero(module_hom_kernel(f))
}

/// The kernel of a bundled module homomorphism is closed under addition.
theorem module_hom_kernel_contains_add[R: Ring, M: AddCommGroup, N: AddCommGroup](
    f: ModuleHom[R, M, N], a: M, b: M
) {
    module_hom_kernel(f).contains(a) and module_hom_kernel(f).contains(b)
        implies module_hom_kernel(f).contains(a + b)
} by {
    submodule_contains_add(module_hom_kernel(f), a, b)
}

/// The kernel of a bundled module homomorphism is closed under negation.
theorem module_hom_kernel_contains_neg[R: Ring, M: AddCommGroup, N: AddCommGroup](
    f: ModuleHom[R, M, N], a: M
) {
    module_hom_kernel(f).contains(a) implies module_hom_kernel(f).contains(-a)
} by {
    submodule_contains_neg(module_hom_kernel(f), a)
}

/// The kernel of a bundled module homomorphism is closed under the scalar action.
theorem module_hom_kernel_contains_smul[R: Ring, M: AddCommGroup, N: AddCommGroup](
    f: ModuleHom[R, M, N], r: R, a: M
) {
    module_hom_kernel(f).contains(a) implies module_hom_kernel(f).contains(f.src.smul(r, a))
} by {
    submodule_contains_smul(module_hom_kernel(f), r, a)
}

/// The kernel of a bundled module homomorphism is contained in the full source submodule.
theorem module_hom_kernel_subset_full[R: Ring, M: AddCommGroup, N: AddCommGroup](
    f: ModuleHom[R, M, N]
) {
    submodule_subset(module_hom_kernel(f), full_submodule(f.src))
} by {
    module_hom_kernel_carrier(f)
    submodule_subset_full(module_hom_kernel(f))
}

/// The image of a bundled module homomorphism contains the zero of the destination module.
theorem module_hom_image_contains_zero[R: Ring, M: AddCommGroup, N: AddCommGroup](
    f: ModuleHom[R, M, N]
) {
    module_hom_image(f).contains(N.0)
} by {
    submodule_contains_zero(module_hom_image(f))
}

/// The image of a bundled module homomorphism is closed under addition.
theorem module_hom_image_contains_add[R: Ring, M: AddCommGroup, N: AddCommGroup](
    f: ModuleHom[R, M, N], a: N, b: N
) {
    module_hom_image(f).contains(a) and module_hom_image(f).contains(b)
        implies module_hom_image(f).contains(a + b)
} by {
    submodule_contains_add(module_hom_image(f), a, b)
}

/// The image of a bundled module homomorphism is closed under negation.
theorem module_hom_image_contains_neg[R: Ring, M: AddCommGroup, N: AddCommGroup](
    f: ModuleHom[R, M, N], a: N
) {
    module_hom_image(f).contains(a) implies module_hom_image(f).contains(-a)
} by {
    submodule_contains_neg(module_hom_image(f), a)
}

/// The image of a bundled module homomorphism is closed under the scalar action.
theorem module_hom_image_contains_smul[R: Ring, M: AddCommGroup, N: AddCommGroup](
    f: ModuleHom[R, M, N], r: R, a: N
) {
    module_hom_image(f).contains(a) implies module_hom_image(f).contains(f.dst.smul(r, a))
} by {
    submodule_contains_smul(module_hom_image(f), r, a)
}

/// The image of a bundled module homomorphism is contained in the full destination submodule.
theorem module_hom_image_subset_full[R: Ring, M: AddCommGroup, N: AddCommGroup](
    f: ModuleHom[R, M, N]
) {
    submodule_subset(module_hom_image(f), full_submodule(f.dst))
} by {
    module_hom_image_carrier(f)
    submodule_subset_full(module_hom_image(f))
}
