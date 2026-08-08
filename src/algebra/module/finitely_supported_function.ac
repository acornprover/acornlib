/// A bundled wrapper for finitely supported functions.

from algebra.add import Add
from algebra.add_semigroup import AddSemigroup
from algebra.add_comm_semigroup import AddCommSemigroup
from algebra.add_monoid import AddMonoid
from algebra.add_comm_monoid import AddCommMonoid
from algebra.add_group import AddGroup
from algebra.add_comm_group import AddCommGroup
from finite_set import FiniteSet
from algebra.module.finite_support import has_support_in, has_support_in_apply_zero, is_finitely_supported,
    pointwise_add_is_finitely_supported, pointwise_neg_is_finitely_supported
from data.basic.function_algebra import pointwise_add, pointwise_zero, pointwise_neg
from data.basic.functions import function_extensionality
from algebra.neg import Neg
from algebra.zero import Zero

/// A function `T -> A` bundled with the fact that it has finite support.
structure FinitelySupportedFunction[T, A: AddCommGroup] {
    /// The underlying function.
    to_fun: T -> A
} constraint {
    is_finitely_supported(to_fun)
}

/// Finitely supported functions are extensional in their underlying functions.
theorem finitely_supported_function_ext[T, A: AddCommGroup](
    f: FinitelySupportedFunction[T, A],
    g: FinitelySupportedFunction[T, A]
) {
    f.to_fun = g.to_fun implies f = g
} by {
    f.to_fun = g.to_fun
}

/// Pointwise equality determines equality of finitely supported functions.
theorem finitely_supported_function_ext_pointwise[T, A: AddCommGroup](
    f: FinitelySupportedFunction[T, A],
    g: FinitelySupportedFunction[T, A]
) {
    (forall(t: T) { f.to_fun(t) = g.to_fun(t) }) implies f = g
} by {
    if forall(t: T) { f.to_fun(t) = g.to_fun(t) } {
        function_extensionality(f.to_fun, g.to_fun)
        f.to_fun = g.to_fun
        finitely_supported_function_ext(f, g)
    }
}

/// The underlying function of a bundled finitely supported function is finitely supported.
theorem finitely_supported_function_is_finitely_supported[T, A: AddCommGroup](
    f: FinitelySupportedFunction[T, A]
) {
    is_finitely_supported(f.to_fun)
}

/// A chosen finite support for a bundled finitely supported function.
let finitely_supported_function_support[T, A: AddCommGroup](f: FinitelySupportedFunction[T, A]) -> result: FiniteSet[T] satisfy {
    has_support_in(f.to_fun, result)
} by {
}

/// The chosen support supports the underlying function.
theorem finitely_supported_function_support_has_support_in[T, A: AddCommGroup](
    f: FinitelySupportedFunction[T, A]
) {
    has_support_in(f.to_fun, finitely_supported_function_support(f))
} by {
}

/// A bundled function is zero outside its chosen support.
theorem finitely_supported_function_apply_zero_outside_support[T, A: AddCommGroup](
    f: FinitelySupportedFunction[T, A],
    t: T
) {
    not finitely_supported_function_support(f).contains(t) implies f.to_fun(t) = A.0
} by {
    if not finitely_supported_function_support(f).contains(t) {
        finitely_supported_function_support_has_support_in(f)
        has_support_in_apply_zero[T, A](f.to_fun, finitely_supported_function_support(f), t)
    }
}

/// A bundled finite support gives zero outside the support.
theorem finitely_supported_function_apply_zero_of_not_contains[T, A: AddCommGroup](
    f: FinitelySupportedFunction[T, A],
    s: FiniteSet[T],
    t: T
) {
    has_support_in(f.to_fun, s) and not s.contains(t) implies f.to_fun(t) = A.0
} by {
    has_support_in_apply_zero[T, A](f.to_fun, s, t)
}

/// The zero finitely supported function.
let finitely_supported_function_zero[T, A: AddCommGroup]: FinitelySupportedFunction[T, A] satisfy {
    FinitelySupportedFunction[T, A].new(pointwise_zero[T, A]) = Option.some(finitely_supported_function_zero)
}

/// The pointwise sum of two finitely supported functions.
let finitely_supported_function_add[T, A: AddCommGroup](f: FinitelySupportedFunction[T, A], g: FinitelySupportedFunction[T, A]) -> result: FinitelySupportedFunction[T, A] satisfy {
    FinitelySupportedFunction[T, A].new(pointwise_add(f.to_fun, g.to_fun)) = Option.some(result)
} by {
    is_finitely_supported(f.to_fun)
    is_finitely_supported(g.to_fun)
    pointwise_add_is_finitely_supported[T, A](f.to_fun, g.to_fun)
}

/// The pointwise additive inverse of a finitely supported function.
let finitely_supported_function_neg[T, A: AddCommGroup](f: FinitelySupportedFunction[T, A]) -> result: FinitelySupportedFunction[T, A] satisfy {
    FinitelySupportedFunction[T, A].new(pointwise_neg(f.to_fun)) = Option.some(result)
} by {
    is_finitely_supported(f.to_fun)
    pointwise_neg_is_finitely_supported[T, A](f.to_fun)
}

attributes FinitelySupportedFunction[T, A: AddCommGroup] {
    /// Extensionality for finitely supported functions.
    let ext = finitely_supported_function_ext[T, A]

    /// A chosen finite support for the underlying function.
    define support(self) -> FiniteSet[T] {
        finitely_supported_function_support(self)
    }

    /// The zero finitely supported function.
    let zero: FinitelySupportedFunction[T, A] = finitely_supported_function_zero[T, A]

    /// Pointwise addition of finitely supported functions.
    define add(self, other: FinitelySupportedFunction[T, A]) -> FinitelySupportedFunction[T, A] {
        finitely_supported_function_add(self, other)
    }

    /// Pointwise additive inverse of a finitely supported function.
    define neg(self) -> FinitelySupportedFunction[T, A] {
        finitely_supported_function_neg(self)
    }

    /// Pointwise difference of finitely supported functions.
    define sub(self, other: FinitelySupportedFunction[T, A]) -> FinitelySupportedFunction[T, A] {
        self + other.neg
    }
}

/// Finitely supported functions have a zero element.
instance FinitelySupportedFunction[T, A: AddCommGroup]: Zero {
    let 0 = FinitelySupportedFunction[T, A].zero
}

/// Finitely supported functions have pointwise addition.
instance FinitelySupportedFunction[T, A: AddCommGroup]: Add {
    let add = FinitelySupportedFunction[T, A].add
}

/// Finitely supported functions have pointwise additive inverses.
instance FinitelySupportedFunction[T, A: AddCommGroup]: Neg {
    let neg = FinitelySupportedFunction[T, A].neg
}

/// The zero finitely supported function evaluates to zero.
theorem finitely_supported_function_zero_apply[T, A: AddCommGroup](t: T) {
    FinitelySupportedFunction[T, A].zero.to_fun(t) = A.0
} by {
    finitely_supported_function_zero[T, A].to_fun = pointwise_zero[T, A]
    pointwise_zero[T, A](t) = A.0
}

/// The zero bundled function has the pointwise-zero underlying function.
theorem finitely_supported_function_zero_to_fun[T, A: AddCommGroup] {
    FinitelySupportedFunction[T, A].zero.to_fun = pointwise_zero[T, A]
} by {
}

/// The pointwise sum evaluates to the sum of values.
theorem finitely_supported_function_add_apply[T, A: AddCommGroup](
    f: FinitelySupportedFunction[T, A],
    g: FinitelySupportedFunction[T, A],
    t: T
) {
    (f + g).to_fun(t) = f.to_fun(t) + g.to_fun(t)
} by {
    finitely_supported_function_add(f, g).to_fun = pointwise_add(f.to_fun, g.to_fun)
}

/// The bundled sum has the pointwise-sum underlying function.
theorem finitely_supported_function_add_to_fun[T, A: AddCommGroup](
    f: FinitelySupportedFunction[T, A],
    g: FinitelySupportedFunction[T, A]
) {
    (f + g).to_fun = pointwise_add(f.to_fun, g.to_fun)
} by {
}

/// The pointwise additive inverse evaluates to the inverse of the value.
theorem finitely_supported_function_neg_apply[T, A: AddCommGroup](
    f: FinitelySupportedFunction[T, A],
    t: T
) {
    (-f).to_fun(t) = -f.to_fun(t)
} by {
    finitely_supported_function_neg(f).to_fun = pointwise_neg(f.to_fun)
}

/// The bundled negative has the pointwise-negative underlying function.
theorem finitely_supported_function_neg_to_fun[T, A: AddCommGroup](
    f: FinitelySupportedFunction[T, A]
) {
    (-f).to_fun = pointwise_neg(f.to_fun)
} by {
}

/// The pointwise difference evaluates to the difference of values.
theorem finitely_supported_function_sub_apply[T, A: AddCommGroup](
    f: FinitelySupportedFunction[T, A],
    g: FinitelySupportedFunction[T, A],
    t: T
) {
    f.sub(g).to_fun(t) = f.to_fun(t) + -g.to_fun(t)
} by {
    finitely_supported_function_add_apply(f, g.neg, t)
    finitely_supported_function_neg_apply(g, t)
}

/// The bundled difference has the expected pointwise-difference underlying function.
theorem finitely_supported_function_sub_to_fun[T, A: AddCommGroup](
    f: FinitelySupportedFunction[T, A],
    g: FinitelySupportedFunction[T, A]
) {
    f.sub(g).to_fun = pointwise_add(f.to_fun, pointwise_neg(g.to_fun))
} by {
    forall(t: T) {
        finitely_supported_function_sub_apply(f, g, t)
        f.sub(g).to_fun(t) = pointwise_add(f.to_fun, pointwise_neg(g.to_fun), t)
    }
    function_extensionality(f.sub(g).to_fun, pointwise_add(f.to_fun, pointwise_neg(g.to_fun)))
}

/// Pointwise addition of finitely supported functions is associative.
theorem finitely_supported_function_add_semigroup_law[T, A: AddCommGroup](
    f: FinitelySupportedFunction[T, A],
    g: FinitelySupportedFunction[T, A],
    h: FinitelySupportedFunction[T, A]
) {
    Add.add(f, Add.add(g, h)) = Add.add(Add.add(f, g), h)
} by {
    forall(t: T) {
        finitely_supported_function_add_apply(g, h, t)
        finitely_supported_function_add_apply(f, g + h, t)
        finitely_supported_function_add_apply(f, g, t)
        finitely_supported_function_add_apply(f + g, h, t)
        f.to_fun(t) + (g.to_fun(t) + h.to_fun(t)) =
            (f.to_fun(t) + g.to_fun(t)) + h.to_fun(t)
        (f + (g + h)).to_fun(t) = ((f + g) + h).to_fun(t)
    }
    finitely_supported_function_ext_pointwise(f + (g + h), (f + g) + h)
}

/// Pointwise addition of finitely supported functions is commutative.
theorem finitely_supported_function_add_comm_semigroup_law[T, A: AddCommGroup](
    f: FinitelySupportedFunction[T, A],
    g: FinitelySupportedFunction[T, A]
) {
    Add.add(f, g) = Add.add(g, f)
} by {
    forall(t: T) {
        finitely_supported_function_add_apply(f, g, t)
        finitely_supported_function_add_apply(g, f, t)
        f.to_fun(t) + g.to_fun(t) = g.to_fun(t) + f.to_fun(t)
        (f + g).to_fun(t) = (g + f).to_fun(t)
    }
    finitely_supported_function_ext_pointwise(f + g, g + f)
}

/// Zero is a right identity for pointwise addition of finitely supported functions.
theorem finitely_supported_function_add_monoid_right_law[T, A: AddCommGroup](
    f: FinitelySupportedFunction[T, A]
) {
    Add.add(f, Zero.0[FinitelySupportedFunction[T, A]]) = f
} by {
    forall(t: T) {
        finitely_supported_function_add_apply(f, FinitelySupportedFunction[T, A].zero, t)
        finitely_supported_function_zero_apply[T, A](t)
        (f + FinitelySupportedFunction[T, A].zero).to_fun(t) = f.to_fun(t)
    }
    finitely_supported_function_ext_pointwise(f + FinitelySupportedFunction[T, A].zero, f)
}

/// Zero is a left identity for pointwise addition of finitely supported functions.
theorem finitely_supported_function_add_monoid_left_law[T, A: AddCommGroup](
    f: FinitelySupportedFunction[T, A]
) {
    Add.add(Zero.0[FinitelySupportedFunction[T, A]], f) = f
} by {
    forall(t: T) {
        finitely_supported_function_add_apply(FinitelySupportedFunction[T, A].zero, f, t)
        finitely_supported_function_zero_apply[T, A](t)
        (FinitelySupportedFunction[T, A].zero + f).to_fun(t) = f.to_fun(t)
    }
    finitely_supported_function_ext_pointwise(FinitelySupportedFunction[T, A].zero + f, f)
}

/// Pointwise additive inverse is a right inverse for finitely supported functions.
theorem finitely_supported_function_add_group_inverse_law[T, A: AddCommGroup](
    f: FinitelySupportedFunction[T, A]
) {
    Add.add(f, Neg.neg(f)) = Zero.0[FinitelySupportedFunction[T, A]]
} by {
    forall(t: T) {
        finitely_supported_function_add_apply(f, -f, t)
        finitely_supported_function_neg_apply(f, t)
        f.to_fun(t) + -f.to_fun(t) = A.0
        finitely_supported_function_zero_apply[T, A](t)
        (f + -f).to_fun(t) = FinitelySupportedFunction[T, A].zero.to_fun(t)
    }
    finitely_supported_function_ext_pointwise(f + -f, FinitelySupportedFunction[T, A].zero)
}

/// Finitely supported functions form an additive semigroup.
instance FinitelySupportedFunction[T, A: AddCommGroup]: AddSemigroup

/// Finitely supported functions form an additive commutative semigroup.
instance FinitelySupportedFunction[T, A: AddCommGroup]: AddCommSemigroup

/// Finitely supported functions form an additive monoid.
instance FinitelySupportedFunction[T, A: AddCommGroup]: AddMonoid

/// Finitely supported functions form an additive commutative monoid.
instance FinitelySupportedFunction[T, A: AddCommGroup]: AddCommMonoid

/// Finitely supported functions form an additive group.
instance FinitelySupportedFunction[T, A: AddCommGroup]: AddGroup

/// Finitely supported functions form an additive commutative group.
instance FinitelySupportedFunction[T, A: AddCommGroup]: AddCommGroup
