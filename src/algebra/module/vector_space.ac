from algebra.field.field import Field
from algebra.add_comm_group import AddCommGroup
from algebra.module.module import Module, module_smul_zero_left, module_smul_zero_right, module_smul_assoc, module_smul_one, module_smul_neg_left, module_smul_add_left, module_smul_neg_right
from algebra.module.submodule import is_submodule, submodule_zero_constraint, submodule_add_constraint,
    submodule_neg_constraint, submodule_smul_constraint, submodule_closure_is_submodule,
    submodule_closure_contains_eq, submodule_closure_contains, submodule_closure
from data.basic.set import Set
from data.basic.functions import predicate_extensionality

/// Convention: a vector space over a field F is a Module[F, V] with V: AddCommGroup.
/// We do not introduce a separate VectorSpace structure; instead we re-use the
/// existing Module API and add field-specific lemmas here. A subspace of a vector
/// space is a Submodule (see algebra/module/submodule.ac); the subspace criterion,
/// the span construction, and linear independence of singletons are developed here.

/// Scalar multiplication by a nonzero element annihilates only the zero vector.
theorem vector_space_smul_eq_zero[F: Field, V: AddCommGroup](
    m: Module[F, V], r: F, v: V) {
    m.smul(r, v) = V.0 implies r = F.0 or v = V.0
} by {
    if m.smul(r, v) = V.0 {
        if r = F.0 {
            r = F.0 or v = V.0
        } else {
            r * r.inverse = F.1
            m.smul(r * r.inverse, v) = m.smul(F.1, v)
            m.smul(r * r.inverse, v) = v
            m.smul(r, m.smul(r.inverse, v)) = m.smul(r * r.inverse, v)
            m.smul(r * r.inverse, v) = m.smul(r, m.smul(r.inverse, v))
            v = m.smul(r, m.smul(r.inverse, v))
            // Use commutativity of ring multiplication.
            r.inverse * r = r * r.inverse
            r.inverse * r = F.1
            m.smul(r.inverse * r, v) = m.smul(F.1, v)
            m.smul(r.inverse * r, v) = v
            m.smul(r.inverse, m.smul(r, v)) = m.smul(r.inverse * r, v)
            m.smul(r.inverse, m.smul(r, v)) = v
            m.smul(r.inverse, V.0) = v
            module_smul_zero_right(m, r.inverse)
            m.smul(r.inverse, V.0) = V.0
            v = V.0
            r = F.0 or v = V.0
        }
    }
}

/// Scalar multiplication of nonzero scalar by nonzero vector is nonzero.
theorem vector_space_smul_ne_zero[F: Field, V: AddCommGroup](
    m: Module[F, V], r: F, v: V) {
    r != F.0 and v != V.0 implies m.smul(r, v) != V.0
} by {
    if r != F.0 and v != V.0 {
        if m.smul(r, v) = V.0 {
            vector_space_smul_eq_zero(m, r, v)
            false
        }
        m.smul(r, v) != V.0
    }
}

/// Multiplying by the inverse undoes scalar multiplication, on the left.
theorem vector_space_smul_inverse_left[F: Field, V: AddCommGroup](
    m: Module[F, V], r: F, v: V) {
    r != F.0 implies m.smul(r.inverse, m.smul(r, v)) = v
} by {
    if r != F.0 {
        r.inverse * r = F.1
        module_smul_assoc(m, r.inverse, r, v)
        m.smul(r.inverse, m.smul(r, v)) = m.smul(r.inverse * r, v)
        m.smul(r.inverse * r, v) = m.smul(F.1, v)
        module_smul_one(m, v)
        m.smul(r.inverse, m.smul(r, v)) = v
    }
}

/// Multiplying by the inverse undoes scalar multiplication, on the right.
theorem vector_space_smul_inverse_right[F: Field, V: AddCommGroup](
    m: Module[F, V], r: F, v: V) {
    r != F.0 implies m.smul(r, m.smul(r.inverse, v)) = v
} by {
    if r != F.0 {
        r * r.inverse = F.1
        module_smul_assoc(m, r, r.inverse, v)
        m.smul(r, m.smul(r.inverse, v)) = m.smul(r * r.inverse, v)
        m.smul(r * r.inverse, v) = m.smul(F.1, v)
        module_smul_one(m, v)
        m.smul(r, m.smul(r.inverse, v)) = v
    }
}

/// Left cancellation: a nonzero scalar can be cancelled from scalar multiplication.
theorem vector_space_smul_left_cancel[F: Field, V: AddCommGroup](
    m: Module[F, V], r: F, u: V, v: V) {
    r != F.0 and m.smul(r, u) = m.smul(r, v) implies u = v
} by {
    if r != F.0 and m.smul(r, u) = m.smul(r, v) {
        m.smul(r.inverse, m.smul(r, u)) = m.smul(r.inverse, m.smul(r, v))
        vector_space_smul_inverse_left(m, r, u)
        m.smul(r.inverse, m.smul(r, u)) = u
        vector_space_smul_inverse_left(m, r, v)
        m.smul(r.inverse, m.smul(r, v)) = v
        u = v
    }
}

/// Right cancellation: a nonzero vector forces scalar agreement.
theorem vector_space_smul_right_cancel[F: Field, V: AddCommGroup](
    m: Module[F, V], r: F, s: F, v: V) {
    v != V.0 and m.smul(r, v) = m.smul(s, v) implies r = s
} by {
    if v != V.0 and m.smul(r, v) = m.smul(s, v) {
        m.smul(r, v) + -m.smul(s, v) = m.smul(s, v) + -m.smul(s, v)
        m.smul(s, v) + -m.smul(s, v) = V.0
        m.smul(r, v) + -m.smul(s, v) = V.0
        module_smul_neg_left(m, s, v)
        -m.smul(s, v) = m.smul(-s, v)
        m.smul(r, v) + m.smul(-s, v) = V.0
        module_smul_add_left(m, r, -s, v)
        m.smul(r + -s, v) = m.smul(r, v) + m.smul(-s, v)
        m.smul(r + -s, v) = V.0
        vector_space_smul_eq_zero(m, r + -s, v)
        r + -s = F.0 or v = V.0
        r + -s = F.0
        r + -s + s = F.0 + s
        r + (-s + s) = F.0 + s
        -s + s = F.0
        r + F.0 = F.0 + s
        r = s
    }
}

/// Solving for a vector: from `m.smul(r, v) = u` recover `v` via the inverse scalar.
theorem vector_space_smul_eq_implies_eq_inverse_smul[F: Field, V: AddCommGroup](
    m: Module[F, V], r: F, u: V, v: V) {
    r != F.0 and m.smul(r, v) = u implies v = m.smul(r.inverse, u)
} by {
    if r != F.0 and m.smul(r, v) = u {
        m.smul(r.inverse, m.smul(r, v)) = m.smul(r.inverse, u)
        vector_space_smul_inverse_left(m, r, v)
        v = m.smul(r.inverse, u)
    }
}

/// Solving for a vector: `v = m.smul(r.inverse, u)` implies `m.smul(r, v) = u`.
theorem vector_space_inverse_smul_implies_smul_eq[F: Field, V: AddCommGroup](
    m: Module[F, V], r: F, u: V, v: V) {
    r != F.0 and v = m.smul(r.inverse, u) implies m.smul(r, v) = u
} by {
    if r != F.0 and v = m.smul(r.inverse, u) {
        m.smul(r, v) = m.smul(r, m.smul(r.inverse, u))
        vector_space_smul_inverse_right(m, r, u)
        m.smul(r, v) = u
    }
}

/// Scalar multiplication preserves equality of vectors.
theorem vector_space_smul_eq_iff_left_cancel[F: Field, V: AddCommGroup](
    m: Module[F, V], r: F, u: V, v: V) {
    r != F.0 implies (m.smul(r, u) = m.smul(r, v) iff u = v)
} by {
    if r != F.0 {
        if m.smul(r, u) = m.smul(r, v) {
            vector_space_smul_left_cancel(m, r, u, v)
            u = v
        }
        if u = v {
            m.smul(r, u) = m.smul(r, v)
        }
    }
}

/// Scalar multiplication is zero iff the scalar or the vector is zero.
theorem vector_space_smul_eq_zero_iff[F: Field, V: AddCommGroup](
    m: Module[F, V], r: F, v: V) {
    m.smul(r, v) = V.0 iff (r = F.0 or v = V.0)
} by {
    if m.smul(r, v) = V.0 {
        vector_space_smul_eq_zero(m, r, v)
        r = F.0 or v = V.0
    }
    if r = F.0 or v = V.0 {
        if r = F.0 {
            module_smul_zero_left(m, v)
            m.smul(F.0, v) = V.0
            m.smul(r, v) = V.0
        }
        if v = V.0 {
            module_smul_zero_right(m, r)
            m.smul(r, V.0) = V.0
            m.smul(r, v) = V.0
        }
        m.smul(r, v) = V.0
    }
}

/// The negation of a scalar-multiplied vector equals scalar multiplication by the negated vector.
theorem vector_space_neg_smul_eq_smul_neg[F: Field, V: AddCommGroup](
    m: Module[F, V], r: F, v: V) {
    -m.smul(r, v) = m.smul(r, -v)
} by {
    module_smul_neg_right(m, r, v)
}

/// Scalar multiplication on a nonzero vector is injective in the scalar.
theorem vector_space_smul_eq_iff_right_cancel[F: Field, V: AddCommGroup](
    m: Module[F, V], r: F, s: F, v: V) {
    v != V.0 implies (m.smul(r, v) = m.smul(s, v) iff r = s)
} by {
    if v != V.0 {
        if m.smul(r, v) = m.smul(s, v) {
            vector_space_smul_right_cancel(m, r, s, v)
            r = s
        }
        if r = s {
            m.smul(r, v) = m.smul(s, v)
        }
    }
}

/// Scalar multiplication is nonzero iff both the scalar and the vector are nonzero.
theorem vector_space_smul_ne_zero_iff[F: Field, V: AddCommGroup](
    m: Module[F, V], r: F, v: V) {
    m.smul(r, v) != V.0 iff (r != F.0 and v != V.0)
} by {
    vector_space_smul_eq_zero_iff(m, r, v)
}

/// Solving for a vector via the inverse scalar is reversible.
theorem vector_space_smul_eq_iff_eq_inverse_smul[F: Field, V: AddCommGroup](
    m: Module[F, V], r: F, u: V, v: V) {
    r != F.0 implies (m.smul(r, v) = u iff v = m.smul(r.inverse, u))
} by {
    if r != F.0 {
        if m.smul(r, v) = u {
            vector_space_smul_eq_implies_eq_inverse_smul(m, r, u, v)
            v = m.smul(r.inverse, u)
        }
        if v = m.smul(r.inverse, u) {
            vector_space_inverse_smul_implies_smul_eq(m, r, u, v)
            m.smul(r, v) = u
        }
    }
}

/// The zero scalar annihilates every vector.
theorem vector_space_smul_zero[F: Field, V: AddCommGroup](m: Module[F, V], v: V) {
    m.smul(F.0, v) = V.0
} by {
    module_smul_zero_left(m, v)
}

/// The identity scalar acts as the identity on every vector.
theorem vector_space_smul_one[F: Field, V: AddCommGroup](m: Module[F, V], v: V) {
    m.smul(F.1, v) = v
} by {
    module_smul_one(m, v)
}

/// Scaling by the additive inverse of one negates the vector.
theorem vector_space_smul_neg_one[F: Field, V: AddCommGroup](m: Module[F, V], v: V) {
    m.smul(-F.1, v) = -v
} by {
    module_smul_neg_left(m, F.1, v)
    m.smul(-F.1, v) = -m.smul(F.1, v)
    module_smul_one(m, v)
    m.smul(F.1, v) = v
    -m.smul(F.1, v) = -v
    m.smul(-F.1, v) = -v
}

/// A nonempty subset of a vector space closed under addition and scalar multiplication is a subspace.
theorem vector_space_subspace_criterion[F: Field, V: AddCommGroup](
    m: Module[F, V], contains: V -> Bool
) {
    (exists(x: V) { contains(x) })
    and (forall(a: V, b: V) { contains(a) and contains(b) implies contains(a + b) })
    and (forall(r: F, a: V) { contains(a) implies contains(m.smul(r, a)) })
    implies is_submodule(m, contains)
} by {
    if exists(x: V) { contains(x) }
        and (forall(a: V, b: V) { contains(a) and contains(b) implies contains(a + b) })
        and (forall(r: F, a: V) { contains(a) implies contains(m.smul(r, a)) }) {
        let x0: V satisfy {
            contains(x0)
        }
        // The subset contains the zero vector, since 0·x0 = 0.
        submodule_smul_constraint(m, contains) = forall(q: F, y: V) {
            contains(y) implies contains(m.smul(q, y))
        }
        contains(m.smul(F.0, x0))
        module_smul_zero_left(m, x0)
        m.smul(F.0, x0) = V.0
        contains(V.0)
        submodule_zero_constraint(m, contains) = contains(V.0)
        submodule_zero_constraint(m, contains)

        // The subset is closed under addition by hypothesis.
        submodule_add_constraint(m, contains) = forall(a: V, b: V) {
            contains(a) and contains(b) implies contains(a + b)
        }
        submodule_add_constraint(m, contains)

        // The subset is closed under negation, since (-1)·a = -a.
        forall(a: V) {
            if contains(a) {
                contains(m.smul(-F.1, a))
                module_smul_neg_left(m, F.1, a)
                m.smul(-F.1, a) = -m.smul(F.1, a)
                module_smul_one(m, a)
                m.smul(F.1, a) = a
                -m.smul(F.1, a) = -a
                m.smul(-F.1, a) = -a
                contains(-a)
            }
        }
        submodule_neg_constraint(m, contains) = forall(a: V) {
            contains(a) implies contains(-a)
        }
        submodule_neg_constraint(m, contains)

        // The subset is closed under scalar multiplication by hypothesis.
        submodule_smul_constraint(m, contains)

        is_submodule(m, contains) =
            (submodule_zero_constraint(m, contains)
             and submodule_add_constraint(m, contains)
             and submodule_neg_constraint(m, contains)
             and submodule_smul_constraint(m, contains))
        is_submodule(m, contains)
    }
}

/// The span of a set in a vector space is a subspace.
theorem vector_space_span_is_subspace[F: Field, V: AddCommGroup](m: Module[F, V], a: Set[V]) {
    is_submodule(m, submodule_closure(m, a).contains)
} by {
    submodule_closure_is_submodule(m, a)
    is_submodule(m, submodule_closure_contains(m, a))
    forall(x: V) {
        submodule_closure_contains_eq(m, a, x)
        submodule_closure(m, a).contains(x) = submodule_closure_contains(m, a, x)
    }
    predicate_extensionality(submodule_closure(m, a).contains, submodule_closure_contains(m, a))
    submodule_closure(m, a).contains = submodule_closure_contains(m, a)
    is_submodule(m, submodule_closure(m, a).contains)
}

/// True if the singleton {v} is linearly independent, i.e. no nonzero scalar annihilates v.
define vector_space_singleton_independent[F: Field, V: AddCommGroup](
    m: Module[F, V], v: V
) -> Bool {
    forall(r: F) {
        m.smul(r, v) = V.0 implies r = F.0
    }
}

/// A singleton {v} with v nonzero is linearly independent.
theorem vector_space_singleton_independent_of_ne_zero[F: Field, V: AddCommGroup](
    m: Module[F, V], v: V
) {
    v != V.0 implies vector_space_singleton_independent(m, v)
} by {
    if v != V.0 {
        forall(r: F) {
            if m.smul(r, v) = V.0 {
                vector_space_smul_eq_zero(m, r, v)
                r = F.0 or v = V.0
                r = F.0
            }
        }
        vector_space_singleton_independent(m, v) = forall(r: F) {
            m.smul(r, v) = V.0 implies r = F.0
        }
        vector_space_singleton_independent(m, v)
    }
}

/// A linearly independent singleton {v} has v nonzero.
theorem vector_space_ne_zero_of_singleton_independent[F: Field, V: AddCommGroup](
    m: Module[F, V], v: V
) {
    vector_space_singleton_independent(m, v) implies v != V.0
} by {
    if vector_space_singleton_independent(m, v) {
        if v = V.0 {
            m.smul(F.1, v) = m.smul(F.1, V.0)
            module_smul_zero_right(m, F.1)
            m.smul(F.1, V.0) = V.0
            m.smul(F.1, v) = V.0
            vector_space_singleton_independent(m, v) = forall(r: F) {
                m.smul(r, v) = V.0 implies r = F.0
            }
            F.1 = F.0
            F.zero_and_one_are_distinct
            F.0 != F.1
            false
        }
        v != V.0
    }
}

/// A singleton {v} is linearly independent iff v is nonzero.
theorem vector_space_singleton_independent_iff_ne_zero[F: Field, V: AddCommGroup](
    m: Module[F, V], v: V
) {
    vector_space_singleton_independent(m, v) iff v != V.0
} by {
    if v != V.0 {
        vector_space_singleton_independent_of_ne_zero(m, v)
        vector_space_singleton_independent(m, v)
    }
    if vector_space_singleton_independent(m, v) {
        vector_space_ne_zero_of_singleton_independent(m, v)
        v != V.0
    }
    vector_space_singleton_independent(m, v) iff v != V.0
}
