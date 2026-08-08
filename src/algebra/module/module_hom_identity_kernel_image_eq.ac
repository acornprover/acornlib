from algebra.ring.ring import Ring
from algebra.add_comm_group import AddCommGroup
from algebra.module.module import Module
from algebra.module.module_hom import module_hom_identity, module_hom_identity_src, module_hom_identity_dst
from algebra.module.submodule import submodule_eq_of_carrier_contains_at_eq,
    zero_submodule, full_submodule, zero_submodule_carrier_eq, full_submodule_carrier_eq,
    module_hom_kernel, module_hom_image, module_hom_kernel_carrier, module_hom_image_carrier,
    module_hom_identity_kernel_contains_eq_zero_submodule,
    module_hom_identity_image_contains_eq_full_submodule

/// The kernel of the identity module homomorphism is the zero submodule.
theorem module_hom_identity_kernel_eq_zero_submodule[R: Ring, M: AddCommGroup](
    m: Module[R, M]
) {
    module_hom_kernel(module_hom_identity(m)) = zero_submodule(m)
} by {
    let ker = module_hom_kernel(module_hom_identity(m))
    let z = zero_submodule(m)
    module_hom_kernel_carrier(module_hom_identity(m))
    module_hom_identity_src(m)
    zero_submodule_carrier_eq(m)
    ker.carrier = z.carrier
    forall(x: M) {
        module_hom_identity_kernel_contains_eq_zero_submodule(m, x)
        ker.contains(x) = z.contains(x)
    }
    submodule_eq_of_carrier_contains_at_eq(ker, z)
    ker = z
}

/// The image of the identity module homomorphism is the full submodule.
theorem module_hom_identity_image_eq_full_submodule[R: Ring, M: AddCommGroup](
    m: Module[R, M]
) {
    module_hom_image(module_hom_identity(m)) = full_submodule(m)
} by {
    let img = module_hom_image(module_hom_identity(m))
    let full = full_submodule(m)
    module_hom_image_carrier(module_hom_identity(m))
    module_hom_identity_dst(m)
    full_submodule_carrier_eq(m)
    img.carrier = full.carrier
    forall(x: M) {
        module_hom_identity_image_contains_eq_full_submodule(m, x)
        img.contains(x) = full.contains(x)
    }
    submodule_eq_of_carrier_contains_at_eq(img, full)
    img = full
}
