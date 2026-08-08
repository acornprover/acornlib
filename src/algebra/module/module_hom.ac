from algebra.ring.ring import Ring
from algebra.add_comm_group import AddCommGroup
from algebra.add_group import left_cancel
from algebra.add_semigroup import add_fn
from algebra.module.module import Module, module_smul_zero_right, module_smul_neg_left, module_smul_one,
    module_smul_add_right, module_smul_neg_right
from data.basic.functions import identity_fn, compose, is_injective_fn, is_surjective_fn,
    is_bijection_fn, is_left_inverse_fn, is_right_inverse_fn, is_two_sided_inverse_fn,
    bijection_fn_is_injective, bijection_fn_is_surjective, injective_fn_eq,
    two_sided_inverse_fn_imp_bijection_fn, compose_bijection_fn,
    compose_identity_left, compose_identity_right, function_extensionality
from data.basic.relation_transport import preserves_binary_op

/// True if a function from M to N preserves addition.
define preserves_add[M: AddCommGroup, N: AddCommGroup](f: M -> N) -> Bool {
    forall(x: M, y: M) {
        f(x + y) = f(x) + f(y)
    }
}

/// True if a function from M to N preserves the scalar action of R.
define preserves_smul[R: Ring, M: AddCommGroup, N: AddCommGroup](
    src: Module[R, M], dst: Module[R, N], f: M -> N) -> Bool {
    forall(r: R, x: M) {
        f(src.smul(r, x)) = dst.smul(r, f(x))
    }
}

/// True if f is an R-linear map from the source R-module to the destination R-module.
define is_linear_map[R: Ring, M: AddCommGroup, N: AddCommGroup](
    src: Module[R, M], dst: Module[R, N], f: M -> N) -> Bool {
    preserves_add(f) and preserves_smul(src, dst, f)
}

/// The trivial linear map sends every element to the zero of the destination module.
let trivial_linear_map[M: AddCommGroup, N: AddCommGroup]: M -> N = function(a: M) {
    N.0
}

/// The trivial linear map preserves addition.
theorem trivial_linear_map_preserves_add[M: AddCommGroup, N: AddCommGroup] {
    preserves_add(trivial_linear_map[M, N])
} by {
    forall(a: M, b: M) {
        trivial_linear_map[M, N](a + b) = trivial_linear_map[M, N](a) + trivial_linear_map[M, N](b)
    }
}

/// The trivial linear map preserves the scalar action.
theorem trivial_linear_map_preserves_smul[R: Ring, M: AddCommGroup, N: AddCommGroup](
    src: Module[R, M], dst: Module[R, N]) {
    preserves_smul(src, dst, trivial_linear_map[M, N])
} by {
    forall(r: R, a: M) {
        module_smul_zero_right(dst, r)
        trivial_linear_map[M, N](src.smul(r, a)) = dst.smul(r, trivial_linear_map[M, N](a))
    }
}

/// The trivial linear map is a linear map.
theorem trivial_linear_map_is_linear_map[R: Ring, M: AddCommGroup, N: AddCommGroup](
    src: Module[R, M], dst: Module[R, N]) {
    is_linear_map(src, dst, trivial_linear_map[M, N])
} by {
    trivial_linear_map_preserves_add[M, N]
    trivial_linear_map_preserves_smul(src, dst)
}

/// Composing the trivial linear map on the left with any function yields the trivial linear map.
theorem compose_trivial_linear_map_left[M: AddCommGroup, N: AddCommGroup, K: AddCommGroup](f: M -> N) {
    compose(trivial_linear_map[N, K], f) = trivial_linear_map[M, K]
} by {
    forall(x: M) {
        trivial_linear_map[N, K](f(x)) = K.0
        compose(trivial_linear_map[N, K], f, x) = trivial_linear_map[M, K](x)
    }
    function_extensionality(compose(trivial_linear_map[N, K], f), trivial_linear_map[M, K])
}

/// The identity function on a module is a linear map from the module to itself.
theorem identity_fn_is_linear_map[R: Ring, M: AddCommGroup](src: Module[R, M]) {
    is_linear_map(src, src, identity_fn[M])
} by {
    forall(a: M, b: M) {
        identity_fn[M](a + b) = identity_fn[M](a) + identity_fn[M](b)
    }
    preserves_add[M, M](identity_fn[M])
    forall(r: R, a: M) {
        identity_fn[M](src.smul(r, a)) = src.smul(r, identity_fn[M](a))
    }
    preserves_smul(src, src, identity_fn[M])
}

/// Composition of two functions preserves addition when each does.
theorem compose_preserves_add[M: AddCommGroup, N: AddCommGroup, K: AddCommGroup](
    f: N -> K, g: M -> N) {
    preserves_add(f) and preserves_add(g) implies preserves_add(compose(f, g))
} by {
    if preserves_add(f) and preserves_add(g) {
        forall(a: M, b: M) {
            compose(f, g)(a + b) = f(g(a + b))
            g(a + b) = g(a) + g(b)
            f(g(a) + g(b)) = f(g(a)) + f(g(b))
            compose(f, g)(a + b) = compose(f, g)(a) + compose(f, g)(b)
        }
    }
}

/// Composition of two functions preserves the scalar action when each does.
theorem compose_preserves_smul[R: Ring, M: AddCommGroup, N: AddCommGroup, K: AddCommGroup](
    a: Module[R, M], b: Module[R, N], c: Module[R, K], f: N -> K, g: M -> N) {
    preserves_smul(a, b, g) and preserves_smul(b, c, f)
        implies preserves_smul(a, c, compose(f, g))
} by {
    if preserves_smul(a, b, g) and preserves_smul(b, c, f) {
        preserves_smul(a, b, g) = forall(r: R, x: M) {
            g(a.smul(r, x)) = b.smul(r, g(x))
        }
        preserves_smul(b, c, f) = forall(r: R, y: N) {
            f(b.smul(r, y)) = c.smul(r, f(y))
        }
        forall(r: R, x: M) {
            f(b.smul(r, g(x))) = c.smul(r, f(g(x)))
            compose(f, g)(a.smul(r, x)) = c.smul(r, compose(f, g)(x))
        }
    }
}

/// A function preserving addition between additive commutative groups sends zero to zero.
theorem preserves_add_zero[M: AddCommGroup, N: AddCommGroup](f: M -> N) {
    preserves_add(f) implies f(M.0) = N.0
} by {
    if preserves_add(f) {
        preserves_add(f) = forall(x: M, y: M) {
            f(x + y) = f(x) + f(y)
        }
        f(M.0) + f(M.0) = f(M.0) + N.0
        left_cancel(f(M.0), f(M.0), N.0)
    }
}

/// A linear map sends zero to zero.
theorem linear_map_zero[R: Ring, M: AddCommGroup, N: AddCommGroup](
    src: Module[R, M], dst: Module[R, N], f: M -> N) {
    is_linear_map(src, dst, f) implies f(M.0) = N.0
} by {
    if is_linear_map(src, dst, f) {
        is_linear_map(src, dst, f) = preserves_add(f) and preserves_smul(src, dst, f)
        preserves_add_zero(f)
    }
}

/// A linear map preserves the scalar action on a single element.
theorem linear_map_smul[R: Ring, M: AddCommGroup, N: AddCommGroup](
    src: Module[R, M], dst: Module[R, N], f: M -> N, r: R, x: M) {
    is_linear_map(src, dst, f) implies f(src.smul(r, x)) = dst.smul(r, f(x))
} by {
    if is_linear_map(src, dst, f) {
        preserves_smul(src, dst, f) = forall(s: R, y: M) {
            f(src.smul(s, y)) = dst.smul(s, f(y))
        }
    }
}

/// A linear map preserves addition on a single pair.
theorem linear_map_add[R: Ring, M: AddCommGroup, N: AddCommGroup](
    src: Module[R, M], dst: Module[R, N], f: M -> N, x: M, y: M) {
    is_linear_map(src, dst, f) implies f(x + y) = f(x) + f(y)
} by {
    if is_linear_map(src, dst, f) {
        preserves_add(f) = forall(a: M, b: M) {
            f(a + b) = f(a) + f(b)
        }
    }
}

/// A linear map preserves negation.
theorem linear_map_neg[R: Ring, M: AddCommGroup, N: AddCommGroup](
    src: Module[R, M], dst: Module[R, N], f: M -> N, x: M) {
    is_linear_map(src, dst, f) implies f(-x) = -f(x)
} by {
    if is_linear_map(src, dst, f) {
        module_smul_neg_left(src, R.1, x)
        module_smul_one(src, x)
        linear_map_smul(src, dst, f, -R.1, x)
        module_smul_neg_left(dst, R.1, f(x))
        module_smul_one(dst, f(x))
        f(-x) = -f(x)
    }
}

/// A linear map preserves subtraction.
theorem linear_map_sub[R: Ring, M: AddCommGroup, N: AddCommGroup](
    src: Module[R, M], dst: Module[R, N], f: M -> N, x: M, y: M) {
    is_linear_map(src, dst, f) implies f(x - y) = f(x) - f(y)
} by {
    if is_linear_map(src, dst, f) {
        linear_map_add(src, dst, f, x, -y)
        linear_map_neg(src, dst, f, y)
        f(x) + f(-y) = f(x) + -f(y)
        f(x - y) = f(x) - f(y)
    }
}

/// Composition of linear maps is linear.
theorem compose_is_linear_map[R: Ring, M: AddCommGroup, N: AddCommGroup, K: AddCommGroup](
    a: Module[R, M], b: Module[R, N], c: Module[R, K], f: N -> K, g: M -> N
) {
    is_linear_map(b, c, f) and is_linear_map(a, b, g)
    implies is_linear_map(a, c, compose(f, g))
} by {
    if is_linear_map(b, c, f) and is_linear_map(a, b, g) {
        is_linear_map(b, c, f) = preserves_add(f) and preserves_smul(b, c, f)
        compose_preserves_add(f, g)
        compose_preserves_smul(a, b, c, f, g)
        preserves_smul(a, c, compose(f, g))
        is_linear_map(a, c, compose(f, g))
    }
}

/// The pointwise negation of a function from M to N.
define neg_fn[M: AddCommGroup, N: AddCommGroup](f: M -> N, x: M) -> N {
    -f(x)
}

/// The pointwise sum of two addition-preserving functions preserves addition.
theorem add_fn_preserves_add[M: AddCommGroup, N: AddCommGroup](f: M -> N, g: M -> N) {
    preserves_add(f) and preserves_add(g) implies preserves_add(add_fn(f, g))
} by {
    if preserves_add(f) and preserves_add(g) {
        forall(x: M, y: M) {
            add_fn(f, g, x + y) = f(x + y) + g(x + y)
            f(x + y) = f(x) + f(y)
            g(x + y) = g(x) + g(y)
            f(x) + (f(y) + (g(x) + g(y))) = f(x) + (g(x) + (f(y) + g(y)))
            add_fn(f, g, x + y) = add_fn(f, g, x) + add_fn(f, g, y)
        }
    }
}

/// The pointwise sum of two scalar-action-preserving functions preserves the scalar action.
theorem add_fn_preserves_smul[R: Ring, M: AddCommGroup, N: AddCommGroup](
    src: Module[R, M], dst: Module[R, N], f: M -> N, g: M -> N) {
    preserves_smul(src, dst, f) and preserves_smul(src, dst, g)
        implies preserves_smul(src, dst, add_fn(f, g))
} by {
    if preserves_smul(src, dst, f) and preserves_smul(src, dst, g) {
        preserves_smul(src, dst, f) = forall(r: R, x: M) {
            f(src.smul(r, x)) = dst.smul(r, f(x))
        }
        preserves_smul(src, dst, g) = forall(r: R, x: M) {
            g(src.smul(r, x)) = dst.smul(r, g(x))
        }
        forall(r: R, x: M) {
            module_smul_add_right(dst, r, f(x), g(x))
            add_fn(f, g, src.smul(r, x)) = dst.smul(r, add_fn(f, g, x))
        }
    }
}

/// The pointwise sum of two linear maps is a linear map.
theorem add_fn_is_linear_map[R: Ring, M: AddCommGroup, N: AddCommGroup](
    src: Module[R, M], dst: Module[R, N], f: M -> N, g: M -> N) {
    is_linear_map(src, dst, f) and is_linear_map(src, dst, g)
        implies is_linear_map(src, dst, add_fn(f, g))
} by {
    if is_linear_map(src, dst, f) and is_linear_map(src, dst, g) {
        is_linear_map(src, dst, f) = preserves_add(f) and preserves_smul(src, dst, f)
        is_linear_map(src, dst, g) = preserves_add(g) and preserves_smul(src, dst, g)
        add_fn_preserves_add(f, g)
        add_fn_preserves_smul(src, dst, f, g)
        preserves_add(add_fn(f, g)) and preserves_smul(src, dst, add_fn(f, g))
        is_linear_map(src, dst, add_fn(f, g))
    }
}

/// The pointwise negation of an addition-preserving function preserves addition.
theorem neg_fn_preserves_add[M: AddCommGroup, N: AddCommGroup](f: M -> N) {
    preserves_add(f) implies preserves_add(neg_fn(f))
} by {
    if preserves_add(f) {
        forall(x: M, y: M) {
            neg_fn(f, x + y) = -f(x + y)
            f(x + y) = f(x) + f(y)
            -(f(x) + f(y)) = -f(x) + -f(y)
            neg_fn(f, x + y) = neg_fn(f, x) + neg_fn(f, y)
        }
    }
}

/// The pointwise negation of a scalar-action-preserving function preserves the scalar action.
theorem neg_fn_preserves_smul[R: Ring, M: AddCommGroup, N: AddCommGroup](
    src: Module[R, M], dst: Module[R, N], f: M -> N) {
    preserves_smul(src, dst, f) implies preserves_smul(src, dst, neg_fn(f))
} by {
    if preserves_smul(src, dst, f) {
        preserves_smul(src, dst, f) = forall(r: R, x: M) {
            f(src.smul(r, x)) = dst.smul(r, f(x))
        }
        forall(r: R, x: M) {
            module_smul_neg_right(dst, r, f(x))
            neg_fn(f, src.smul(r, x)) = dst.smul(r, neg_fn(f, x))
        }
    }
}

/// The pointwise negation of a linear map is a linear map.
theorem neg_fn_is_linear_map[R: Ring, M: AddCommGroup, N: AddCommGroup](
    src: Module[R, M], dst: Module[R, N], f: M -> N) {
    is_linear_map(src, dst, f) implies is_linear_map(src, dst, neg_fn(f))
} by {
    if is_linear_map(src, dst, f) {
        is_linear_map(src, dst, f) = preserves_add(f) and preserves_smul(src, dst, f)
        neg_fn_preserves_add(f)
        neg_fn_preserves_smul(src, dst, f)
        preserves_add(neg_fn(f)) and preserves_smul(src, dst, neg_fn(f))
        is_linear_map(src, dst, neg_fn(f))
    }
}

/// The pointwise difference of two functions from M to N.
define sub_fn[M: AddCommGroup, N: AddCommGroup](f: M -> N, g: M -> N, x: M) -> N {
    f(x) - g(x)
}

/// The pointwise difference equals the pointwise sum with the negation.
theorem sub_fn_eq_add_neg[M: AddCommGroup, N: AddCommGroup](f: M -> N, g: M -> N, x: M) {
    sub_fn(f, g, x) = add_fn(f, neg_fn(g), x)
} by {
    f(x) - g(x) = f(x) + -g(x)
}

/// The pointwise difference of two addition-preserving functions preserves addition.
theorem sub_fn_preserves_add[M: AddCommGroup, N: AddCommGroup](f: M -> N, g: M -> N) {
    preserves_add(f) and preserves_add(g) implies preserves_add(sub_fn(f, g))
} by {
    if preserves_add(f) and preserves_add(g) {
        neg_fn_preserves_add(g)
        add_fn_preserves_add(f, neg_fn(g))
        forall(x: M) {
            sub_fn_eq_add_neg(f, g, x)
            sub_fn(f, g, x) = add_fn(f, neg_fn(g), x)
        }
        sub_fn(f, g) = add_fn(f, neg_fn(g))
    }
}

/// The pointwise difference of two scalar-action-preserving functions preserves the scalar action.
theorem sub_fn_preserves_smul[R: Ring, M: AddCommGroup, N: AddCommGroup](
    src: Module[R, M], dst: Module[R, N], f: M -> N, g: M -> N) {
    preserves_smul(src, dst, f) and preserves_smul(src, dst, g)
        implies preserves_smul(src, dst, sub_fn(f, g))
} by {
    if preserves_smul(src, dst, f) and preserves_smul(src, dst, g) {
        neg_fn_preserves_smul(src, dst, g)
        add_fn_preserves_smul(src, dst, f, neg_fn(g))
        forall(x: M) {
            sub_fn_eq_add_neg(f, g, x)
            sub_fn(f, g, x) = add_fn(f, neg_fn(g), x)
        }
        sub_fn(f, g) = add_fn(f, neg_fn(g))
    }
}

/// The pointwise difference of two linear maps is a linear map.
theorem sub_fn_is_linear_map[R: Ring, M: AddCommGroup, N: AddCommGroup](
    src: Module[R, M], dst: Module[R, N], f: M -> N, g: M -> N) {
    is_linear_map(src, dst, f) and is_linear_map(src, dst, g)
        implies is_linear_map(src, dst, sub_fn(f, g))
} by {
    if is_linear_map(src, dst, f) and is_linear_map(src, dst, g) {
        is_linear_map(src, dst, f) = preserves_add(f) and preserves_smul(src, dst, f)
        is_linear_map(src, dst, g) = preserves_add(g) and preserves_smul(src, dst, g)
        sub_fn_preserves_add(f, g)
        sub_fn_preserves_smul(src, dst, f, g)
        preserves_add(sub_fn(f, g)) and preserves_smul(src, dst, sub_fn(f, g))
        is_linear_map(src, dst, sub_fn(f, g))
    }
}

/// The pointwise sum with the trivial linear map is the original function.
theorem add_fn_trivial_linear_map_right[M: AddCommGroup, N: AddCommGroup](f: M -> N) {
    add_fn(f, trivial_linear_map[M, N]) = f
} by {
    forall(x: M) {
        trivial_linear_map[M, N](x) = N.0
        add_fn(f, trivial_linear_map[M, N], x) = f(x)
    }
}

/// The pointwise sum with the trivial linear map on the left is the original function.
theorem add_fn_trivial_linear_map_left[M: AddCommGroup, N: AddCommGroup](f: M -> N) {
    add_fn(trivial_linear_map[M, N], f) = f
} by {
    forall(x: M) {
        trivial_linear_map[M, N](x) = N.0
        add_fn(trivial_linear_map[M, N], f, x) = f(x)
    }
}

/// The pointwise sum of a function and its pointwise negation is the trivial linear map.
theorem add_fn_neg_fn[M: AddCommGroup, N: AddCommGroup](f: M -> N) {
    add_fn(f, neg_fn(f)) = trivial_linear_map[M, N]
} by {
    forall(x: M) {
        f(x) + -f(x) = N.0
        add_fn(f, neg_fn(f), x) = trivial_linear_map[M, N](x)
    }
}

/// True if f is an R-linear bijection from the source R-module to the destination R-module.
define is_linear_equiv[R: Ring, M: AddCommGroup, N: AddCommGroup](
    src: Module[R, M], dst: Module[R, N], f: M -> N) -> Bool {
    is_linear_map(src, dst, f) and is_bijection_fn(f)
}

/// A linear equivalence is a linear map.
theorem linear_equiv_is_linear_map[R: Ring, M: AddCommGroup, N: AddCommGroup](
    src: Module[R, M], dst: Module[R, N], f: M -> N) {
    is_linear_equiv(src, dst, f) implies is_linear_map(src, dst, f)
}

/// A linear equivalence is a bijection.
theorem linear_equiv_is_bijection[R: Ring, M: AddCommGroup, N: AddCommGroup](
    src: Module[R, M], dst: Module[R, N], f: M -> N) {
    is_linear_equiv(src, dst, f) implies is_bijection_fn(f)
}

/// The identity function on a module is a linear equivalence from the module to itself.
theorem identity_fn_is_linear_equiv[R: Ring, M: AddCommGroup](src: Module[R, M]) {
    is_linear_equiv(src, src, identity_fn[M])
} by {
    identity_fn_is_linear_map(src)
    forall(x: M, y: M) {
        if identity_fn[M](x) = identity_fn[M](y) {
            x = y
        }
    }
    forall(y: M) {
        exists(x: M) {
            identity_fn[M](x) = y
        }
    }
    is_bijection_fn(identity_fn[M])
}

/// Composition of linear equivalences is a linear equivalence.
theorem compose_is_linear_equiv[R: Ring, M: AddCommGroup, N: AddCommGroup, K: AddCommGroup](
    a: Module[R, M], b: Module[R, N], c: Module[R, K], f: N -> K, g: M -> N
) {
    is_linear_equiv(b, c, f) and is_linear_equiv(a, b, g)
    implies is_linear_equiv(a, c, compose(f, g))
} by {
    if is_linear_equiv(b, c, f) and is_linear_equiv(a, b, g) {
        is_linear_equiv(b, c, f) = is_linear_map(b, c, f) and is_bijection_fn(f)
        is_linear_map(b, c, f)
        is_bijection_fn(f)
        compose_is_linear_map(a, b, c, f, g)
        compose_bijection_fn(f, g)
        is_bijection_fn(compose(f, g))
        is_linear_map(a, c, compose(f, g)) and is_bijection_fn(compose(f, g))
        is_linear_equiv(a, c, compose(f, g))
    }
}

/// A two-sided inverse of a linear map preserves addition.
theorem linear_map_two_sided_inverse_preserves_add[R: Ring, M: AddCommGroup, N: AddCommGroup](
    src: Module[R, M], dst: Module[R, N], f: M -> N, g: N -> M) {
    is_linear_map(src, dst, f) and is_two_sided_inverse_fn(f, g) implies preserves_add(g)
} by {
    if is_linear_map(src, dst, f) and is_two_sided_inverse_fn(f, g) {
        is_two_sided_inverse_fn(f, g) = is_left_inverse_fn(f, g) and is_right_inverse_fn(f, g)
        is_left_inverse_fn(f, g) = forall(x: M) {
            g(f(x)) = x
        }
        is_right_inverse_fn(f, g) = forall(y: N) {
            f(g(y)) = y
        }
        forall(y1: N, y2: N) {
            linear_map_add(src, dst, f, g(y1), g(y2))
            f(g(y1) + g(y2)) = f(g(y1)) + f(g(y2))
            g(y1 + y2) = g(y1) + g(y2)
        }
    }
}

/// A two-sided inverse of a linear map preserves the scalar action.
theorem linear_map_two_sided_inverse_preserves_smul[R: Ring, M: AddCommGroup, N: AddCommGroup](
    src: Module[R, M], dst: Module[R, N], f: M -> N, g: N -> M) {
    is_linear_map(src, dst, f) and is_two_sided_inverse_fn(f, g)
        implies preserves_smul(dst, src, g)
} by {
    if is_linear_map(src, dst, f) and is_two_sided_inverse_fn(f, g) {
        is_two_sided_inverse_fn(f, g) = is_left_inverse_fn(f, g) and is_right_inverse_fn(f, g)
        is_left_inverse_fn(f, g) = forall(x: M) {
            g(f(x)) = x
        }
        is_right_inverse_fn(f, g) = forall(y: N) {
            f(g(y)) = y
        }
        forall(r: R, y: N) {
            linear_map_smul(src, dst, f, r, g(y))
            g(dst.smul(r, y)) = src.smul(r, g(y))
        }
    }
}

/// A two-sided inverse of a linear map is itself a linear map in the opposite direction.
theorem linear_map_two_sided_inverse_is_linear_map[R: Ring, M: AddCommGroup, N: AddCommGroup](
    src: Module[R, M], dst: Module[R, N], f: M -> N, g: N -> M) {
    is_linear_map(src, dst, f) and is_two_sided_inverse_fn(f, g)
        implies is_linear_map(dst, src, g)
} by {
    if is_linear_map(src, dst, f) and is_two_sided_inverse_fn(f, g) {
        linear_map_two_sided_inverse_preserves_add(src, dst, f, g)
        linear_map_two_sided_inverse_preserves_smul(src, dst, f, g)
        preserves_smul(dst, src, g)
        is_linear_map(dst, src, g)
    }
}

/// A two-sided inverse of a linear equivalence is a linear equivalence in the opposite direction.
theorem linear_equiv_two_sided_inverse_is_linear_equiv[R: Ring, M: AddCommGroup, N: AddCommGroup](
    src: Module[R, M], dst: Module[R, N], f: M -> N, g: N -> M) {
    is_linear_equiv(src, dst, f) and is_two_sided_inverse_fn(f, g)
        implies is_linear_equiv(dst, src, g)
} by {
    if is_linear_equiv(src, dst, f) and is_two_sided_inverse_fn(f, g) {
        is_linear_equiv(src, dst, f) = is_linear_map(src, dst, f) and is_bijection_fn(f)
        linear_map_two_sided_inverse_is_linear_map(src, dst, f, g)
        is_linear_map(dst, src, g)
        is_two_sided_inverse_fn(f, g) = is_left_inverse_fn(f, g) and is_right_inverse_fn(f, g)
        is_left_inverse_fn(f, g) = forall(x: M) {
            g(f(x)) = x
        }
        is_right_inverse_fn(f, g) = forall(y: N) {
            f(g(y)) = y
        }
        is_right_inverse_fn(g, f)
        two_sided_inverse_fn_imp_bijection_fn(g, f)
        is_linear_map(dst, src, g) and is_bijection_fn(g)
        is_linear_equiv(dst, src, g)
    }
}

/// A linear equivalence is injective.
theorem linear_equiv_is_injective[R: Ring, M: AddCommGroup, N: AddCommGroup](
    src: Module[R, M], dst: Module[R, N], f: M -> N) {
    is_linear_equiv(src, dst, f) implies is_injective_fn(f)
} by {
    if is_linear_equiv(src, dst, f) {
        is_linear_equiv(src, dst, f) = is_linear_map(src, dst, f) and is_bijection_fn(f)
        bijection_fn_is_injective(f)
    }
}

/// A linear equivalence is surjective.
theorem linear_equiv_is_surjective[R: Ring, M: AddCommGroup, N: AddCommGroup](
    src: Module[R, M], dst: Module[R, N], f: M -> N) {
    is_linear_equiv(src, dst, f) implies is_surjective_fn(f)
} by {
    if is_linear_equiv(src, dst, f) {
        is_linear_equiv(src, dst, f) = is_linear_map(src, dst, f) and is_bijection_fn(f)
        bijection_fn_is_surjective(f)
    }
}

/// A linear equivalence sends zero to zero.
theorem linear_equiv_zero[R: Ring, M: AddCommGroup, N: AddCommGroup](
    src: Module[R, M], dst: Module[R, N], f: M -> N) {
    is_linear_equiv(src, dst, f) implies f(M.0) = N.0
} by {
    if is_linear_equiv(src, dst, f) {
        linear_equiv_is_linear_map(src, dst, f)
        linear_map_zero(src, dst, f)
    }
}

/// A linear equivalence preserves addition.
theorem linear_equiv_add[R: Ring, M: AddCommGroup, N: AddCommGroup](
    src: Module[R, M], dst: Module[R, N], f: M -> N, x: M, y: M) {
    is_linear_equiv(src, dst, f) implies f(x + y) = f(x) + f(y)
} by {
    if is_linear_equiv(src, dst, f) {
        linear_equiv_is_linear_map(src, dst, f)
        linear_map_add(src, dst, f, x, y)
    }
}

/// A linear equivalence preserves the scalar action.
theorem linear_equiv_smul[R: Ring, M: AddCommGroup, N: AddCommGroup](
    src: Module[R, M], dst: Module[R, N], f: M -> N, r: R, x: M) {
    is_linear_equiv(src, dst, f) implies f(src.smul(r, x)) = dst.smul(r, f(x))
} by {
    if is_linear_equiv(src, dst, f) {
        linear_equiv_is_linear_map(src, dst, f)
        linear_map_smul(src, dst, f, r, x)
    }
}

/// A linear equivalence preserves negation.
theorem linear_equiv_neg[R: Ring, M: AddCommGroup, N: AddCommGroup](
    src: Module[R, M], dst: Module[R, N], f: M -> N, x: M) {
    is_linear_equiv(src, dst, f) implies f(-x) = -f(x)
} by {
    if is_linear_equiv(src, dst, f) {
        linear_equiv_is_linear_map(src, dst, f)
        linear_map_neg(src, dst, f, x)
    }
}

/// A linear equivalence preserves subtraction.
theorem linear_equiv_sub[R: Ring, M: AddCommGroup, N: AddCommGroup](
    src: Module[R, M], dst: Module[R, N], f: M -> N, x: M, y: M) {
    is_linear_equiv(src, dst, f) implies f(x - y) = f(x) - f(y)
} by {
    if is_linear_equiv(src, dst, f) {
        linear_equiv_is_linear_map(src, dst, f)
        linear_map_sub(src, dst, f, x, y)
    }
}

/// A linear equivalence equates source and destination points via its inverse.
theorem linear_equiv_apply_eq_iff[R: Ring, M: AddCommGroup, N: AddCommGroup](
    src: Module[R, M], dst: Module[R, N], f: M -> N, g: N -> M, x: M, y: N) {
    is_linear_equiv(src, dst, f) and is_two_sided_inverse_fn(f, g)
        implies (f(x) = y iff x = g(y))
} by {
    if is_linear_equiv(src, dst, f) and is_two_sided_inverse_fn(f, g) {
        is_two_sided_inverse_fn(f, g) = is_left_inverse_fn(f, g) and is_right_inverse_fn(f, g)
        is_left_inverse_fn(f, g) = forall(a: M) {
            g(f(a)) = a
        }
        is_right_inverse_fn(f, g) = forall(b: N) {
            f(g(b)) = b
        }
        if f(x) = y {
            x = g(y)
        }
        if x = g(y) {
            f(x) = y
        }
        (f(x) = y) iff (x = g(y))
    }
}

/// Under a linear equivalence, the image equals zero iff the input is zero.
theorem linear_equiv_apply_eq_zero_iff_eq_zero[R: Ring, M: AddCommGroup, N: AddCommGroup](
    src: Module[R, M], dst: Module[R, N], f: M -> N, x: M) {
    is_linear_equiv(src, dst, f) implies (f(x) = N.0 iff x = M.0)
} by {
    if is_linear_equiv(src, dst, f) {
        linear_equiv_zero(src, dst, f)
        linear_equiv_is_injective(src, dst, f)
        if f(x) = N.0 {
            injective_fn_eq(f, x, M.0)
            x = M.0
        }
        if x = M.0 {
            f(x) = N.0
        }
        (f(x) = N.0) iff (x = M.0)
    }
}

/// A two-sided inverse of a linear equivalence sends zero to zero.
theorem linear_equiv_two_sided_inverse_zero[R: Ring, M: AddCommGroup, N: AddCommGroup](
    src: Module[R, M], dst: Module[R, N], f: M -> N, g: N -> M) {
    is_linear_equiv(src, dst, f) and is_two_sided_inverse_fn(f, g) implies g(N.0) = M.0
} by {
    if is_linear_equiv(src, dst, f) and is_two_sided_inverse_fn(f, g) {
        linear_equiv_two_sided_inverse_is_linear_equiv(src, dst, f, g)
        linear_equiv_zero(dst, src, g)
    }
}

/// A two-sided inverse of a linear equivalence preserves addition.
theorem linear_equiv_two_sided_inverse_add[R: Ring, M: AddCommGroup, N: AddCommGroup](
    src: Module[R, M], dst: Module[R, N], f: M -> N, g: N -> M, x: N, y: N) {
    is_linear_equiv(src, dst, f) and is_two_sided_inverse_fn(f, g)
        implies g(x + y) = g(x) + g(y)
} by {
    if is_linear_equiv(src, dst, f) and is_two_sided_inverse_fn(f, g) {
        linear_equiv_two_sided_inverse_is_linear_equiv(src, dst, f, g)
        linear_equiv_add(dst, src, g, x, y)
    }
}

/// A two-sided inverse of a linear equivalence preserves the scalar action.
theorem linear_equiv_two_sided_inverse_smul[R: Ring, M: AddCommGroup, N: AddCommGroup](
    src: Module[R, M], dst: Module[R, N], f: M -> N, g: N -> M, r: R, x: N) {
    is_linear_equiv(src, dst, f) and is_two_sided_inverse_fn(f, g)
        implies g(dst.smul(r, x)) = src.smul(r, g(x))
} by {
    if is_linear_equiv(src, dst, f) and is_two_sided_inverse_fn(f, g) {
        linear_equiv_two_sided_inverse_is_linear_equiv(src, dst, f, g)
        linear_equiv_smul(dst, src, g, r, x)
    }
}

/// A two-sided inverse of a linear equivalence preserves negation.
theorem linear_equiv_two_sided_inverse_neg[R: Ring, M: AddCommGroup, N: AddCommGroup](
    src: Module[R, M], dst: Module[R, N], f: M -> N, g: N -> M, x: N) {
    is_linear_equiv(src, dst, f) and is_two_sided_inverse_fn(f, g)
        implies g(-x) = -g(x)
} by {
    if is_linear_equiv(src, dst, f) and is_two_sided_inverse_fn(f, g) {
        linear_equiv_two_sided_inverse_is_linear_equiv(src, dst, f, g)
        linear_equiv_neg(dst, src, g, x)
    }
}

/// A two-sided inverse of a linear equivalence preserves subtraction.
theorem linear_equiv_two_sided_inverse_sub[R: Ring, M: AddCommGroup, N: AddCommGroup](
    src: Module[R, M], dst: Module[R, N], f: M -> N, g: N -> M, x: N, y: N) {
    is_linear_equiv(src, dst, f) and is_two_sided_inverse_fn(f, g)
        implies g(x - y) = g(x) - g(y)
} by {
    if is_linear_equiv(src, dst, f) and is_two_sided_inverse_fn(f, g) {
        linear_equiv_two_sided_inverse_is_linear_equiv(src, dst, f, g)
        linear_equiv_sub(dst, src, g, x, y)
    }
}

/// A bundled R-linear map between R-modules.
structure ModuleHom[R: Ring, M: AddCommGroup, N: AddCommGroup] {
    /// The source R-module.
    src: Module[R, M]
    /// The destination R-module.
    dst: Module[R, N]
    /// The underlying function.
    hom: M -> N
} constraint {
    is_linear_map(src, dst, hom)
}

/// Module hom extensionality: two module homomorphisms with the same source and destination are equal when they agree on every input.
theorem module_hom_ext[R: Ring, M: AddCommGroup, N: AddCommGroup](
    f: ModuleHom[R, M, N], g: ModuleHom[R, M, N]
) {
    f.src = g.src and f.dst = g.dst and (forall(x: M) { f.hom(x) = g.hom(x) }) implies f = g
} by {
    if f.src = g.src and f.dst = g.dst and forall(x: M) { f.hom(x) = g.hom(x) } {
        f.hom = g.hom
    }
}

/// Equal module homomorphisms have equal sources.
theorem module_hom_eq_src[R: Ring, M: AddCommGroup, N: AddCommGroup](
    f: ModuleHom[R, M, N], g: ModuleHom[R, M, N]
) {
    f = g implies f.src = g.src
}

/// Equal module homomorphisms have equal destinations.
theorem module_hom_eq_dst[R: Ring, M: AddCommGroup, N: AddCommGroup](
    f: ModuleHom[R, M, N], g: ModuleHom[R, M, N]
) {
    f = g implies f.dst = g.dst
}

/// Equal module homomorphisms have equal underlying functions.
theorem module_hom_eq_hom[R: Ring, M: AddCommGroup, N: AddCommGroup](
    f: ModuleHom[R, M, N], g: ModuleHom[R, M, N]
) {
    f = g implies f.hom = g.hom
}

/// Equal module homomorphisms have equal values at every element.
theorem module_hom_eq_hom_at[R: Ring, M: AddCommGroup, N: AddCommGroup](
    f: ModuleHom[R, M, N], g: ModuleHom[R, M, N], x: M
) {
    f = g implies f.hom(x) = g.hom(x)
} by {
    if f = g {
        f.hom(x) = g.hom(x)
    }
}

/// The underlying function of a module homomorphism is linear.
theorem module_hom_is_linear_map[R: Ring, M: AddCommGroup, N: AddCommGroup](
    f: ModuleHom[R, M, N]
) {
    is_linear_map(f.src, f.dst, f.hom)
}

/// A module homomorphism sends zero to zero.
theorem module_hom_zero[R: Ring, M: AddCommGroup, N: AddCommGroup](
    f: ModuleHom[R, M, N]
) {
    f.hom(M.0) = N.0
} by {
    linear_map_zero(f.src, f.dst, f.hom)
}

/// A module homomorphism preserves addition.
theorem module_hom_add[R: Ring, M: AddCommGroup, N: AddCommGroup](
    f: ModuleHom[R, M, N], x: M, y: M
) {
    f.hom(x + y) = f.hom(x) + f.hom(y)
} by {
    is_linear_map(f.src, f.dst, f.hom)
    linear_map_add(f.src, f.dst, f.hom, x, y)
}

/// A module homomorphism preserves the scalar action.
theorem module_hom_smul[R: Ring, M: AddCommGroup, N: AddCommGroup](
    f: ModuleHom[R, M, N], r: R, x: M
) {
    f.hom(f.src.smul(r, x)) = f.dst.smul(r, f.hom(x))
} by {
    linear_map_smul(f.src, f.dst, f.hom, r, x)
}

/// A module homomorphism preserves negation.
theorem module_hom_neg[R: Ring, M: AddCommGroup, N: AddCommGroup](
    f: ModuleHom[R, M, N], x: M
) {
    f.hom(-x) = -f.hom(x)
} by {
    linear_map_neg(f.src, f.dst, f.hom, x)
}

/// A module homomorphism preserves subtraction.
theorem module_hom_sub[R: Ring, M: AddCommGroup, N: AddCommGroup](
    f: ModuleHom[R, M, N], x: M, y: M
) {
    f.hom(x - y) = f.hom(x) - f.hom(y)
} by {
    is_linear_map(f.src, f.dst, f.hom)
    linear_map_sub(f.src, f.dst, f.hom, x, y)
}

/// The identity module homomorphism on an R-module.
let module_hom_identity[R: Ring, M: AddCommGroup](m: Module[R, M]) -> result: ModuleHom[R, M, M] satisfy {
    ModuleHom.new(m, m, identity_fn[M]) = Option.some(result)
} by {
    identity_fn_is_linear_map(m)
}

/// The identity module homomorphism has the identity function as its underlying map.
theorem module_hom_identity_hom[R: Ring, M: AddCommGroup](m: Module[R, M]) {
    module_hom_identity(m).hom = identity_fn[M]
}

/// The identity module homomorphism's source equals the given module.
theorem module_hom_identity_src[R: Ring, M: AddCommGroup](m: Module[R, M]) {
    module_hom_identity(m).src = m
}

/// The identity module homomorphism's destination equals the given module.
theorem module_hom_identity_dst[R: Ring, M: AddCommGroup](m: Module[R, M]) {
    module_hom_identity(m).dst = m
}

/// The identity module homomorphism evaluated at a point returns the point.
theorem module_hom_identity_hom_at[R: Ring, M: AddCommGroup](m: Module[R, M], x: M) {
    module_hom_identity(m).hom(x) = x
} by {
}

/// The trivial bundled module homomorphism from `src` to `dst`: the constant-zero map.
let module_hom_trivial[R: Ring, M: AddCommGroup, N: AddCommGroup](src: Module[R, M], dst: Module[R, N]) -> result: ModuleHom[R, M, N] satisfy {
    ModuleHom.new(src, dst, trivial_linear_map[M, N]) = Option.some(result)
} by {
    trivial_linear_map_is_linear_map(src, dst)
}

/// The trivial bundled module homomorphism has the trivial linear map as its underlying function.
theorem module_hom_trivial_hom[R: Ring, M: AddCommGroup, N: AddCommGroup](
    src: Module[R, M], dst: Module[R, N]
) {
    module_hom_trivial(src, dst).hom = trivial_linear_map[M, N]
}

/// The trivial bundled module homomorphism's source equals the given source module.
theorem module_hom_trivial_src[R: Ring, M: AddCommGroup, N: AddCommGroup](
    src: Module[R, M], dst: Module[R, N]
) {
    module_hom_trivial(src, dst).src = src
}

/// The trivial bundled module homomorphism's destination equals the given destination module.
theorem module_hom_trivial_dst[R: Ring, M: AddCommGroup, N: AddCommGroup](
    src: Module[R, M], dst: Module[R, N]
) {
    module_hom_trivial(src, dst).dst = dst
}

/// The trivial bundled module homomorphism evaluated at any point returns zero.
theorem module_hom_trivial_hom_at[R: Ring, M: AddCommGroup, N: AddCommGroup](
    src: Module[R, M], dst: Module[R, N], x: M
) {
    module_hom_trivial(src, dst).hom(x) = N.0
} by {
}

/// The composition of two module homomorphisms, returning `Option.some` exactly when the source module of the first equals the destination module of the second.
define module_hom_compose[R: Ring, M: AddCommGroup, N: AddCommGroup, K: AddCommGroup](
    f: ModuleHom[R, N, K], g: ModuleHom[R, M, N]
) -> Option[ModuleHom[R, M, K]] {
    ModuleHom[R, M, K].new(g.src, f.dst, compose(f.hom, g.hom))
}

/// When the source module of the first map equals the destination module of the second, the composition is defined.
theorem module_hom_compose_some[R: Ring, M: AddCommGroup, N: AddCommGroup, K: AddCommGroup](
    f: ModuleHom[R, N, K], g: ModuleHom[R, M, N]
) {
    f.src = g.dst implies exists(h: ModuleHom[R, M, K]) {
        module_hom_compose(f, g) = Option.some(h)
    }
} by {
    if f.src = g.dst {
        is_linear_map(f.src, f.dst, f.hom)
        compose_is_linear_map(g.src, g.dst, f.dst, f.hom, g.hom)
        is_linear_map(g.src, f.dst, compose(f.hom, g.hom))
        let h: ModuleHom[R, M, K] satisfy {
            ModuleHom[R, M, K].new(g.src, f.dst, compose(f.hom, g.hom)) = Option.some(h)
        }
        module_hom_compose(f, g) = Option.some(h)
    }
}

/// If the composition is some bundled hom, its source equals the source of the second map.
theorem module_hom_compose_src[R: Ring, M: AddCommGroup, N: AddCommGroup, K: AddCommGroup](
    f: ModuleHom[R, N, K], g: ModuleHom[R, M, N], h: ModuleHom[R, M, K]
) {
    module_hom_compose(f, g) = Option.some(h) implies h.src = g.src
}

/// If the composition is some bundled hom, its destination equals the destination of the first map.
theorem module_hom_compose_dst[R: Ring, M: AddCommGroup, N: AddCommGroup, K: AddCommGroup](
    f: ModuleHom[R, N, K], g: ModuleHom[R, M, N], h: ModuleHom[R, M, K]
) {
    module_hom_compose(f, g) = Option.some(h) implies h.dst = f.dst
}

/// If the composition is some bundled hom, its underlying function is the iterated composition.
theorem module_hom_compose_hom[R: Ring, M: AddCommGroup, N: AddCommGroup, K: AddCommGroup](
    f: ModuleHom[R, N, K], g: ModuleHom[R, M, N], h: ModuleHom[R, M, K]
) {
    module_hom_compose(f, g) = Option.some(h) implies h.hom = compose(f.hom, g.hom)
} by {
    if module_hom_compose(f, g) = Option.some(h) {
        ModuleHom[R, M, K].new(g.src, f.dst, compose(f.hom, g.hom)) = Option.some(h)
    }
}

/// If the composition is some bundled hom, it evaluates pointwise as the iterated application.
theorem module_hom_compose_hom_at[R: Ring, M: AddCommGroup, N: AddCommGroup, K: AddCommGroup](
    f: ModuleHom[R, N, K], g: ModuleHom[R, M, N], h: ModuleHom[R, M, K], x: M
) {
    module_hom_compose(f, g) = Option.some(h) implies h.hom(x) = f.hom(g.hom(x))
} by {
    if module_hom_compose(f, g) = Option.some(h) {
        h.hom(x) = compose(f.hom, g.hom)(x)
    }
}

/// Composing the identity module homomorphism on the destination with f is defined as some bundled hom.
theorem module_hom_compose_identity_left_some[R: Ring, M: AddCommGroup, N: AddCommGroup](f: ModuleHom[R, M, N]) {
    exists(h: ModuleHom[R, M, N]) {
        module_hom_compose(module_hom_identity(f.dst), f) = Option.some(h)
    }
} by {
    module_hom_identity_src(f.dst)
    module_hom_compose_some(module_hom_identity(f.dst), f)
}

/// Composing the identity on the destination with f yields a hom whose underlying function equals f.hom.
theorem module_hom_compose_identity_left_hom[R: Ring, M: AddCommGroup, N: AddCommGroup](f: ModuleHom[R, M, N], h: ModuleHom[R, M, N]) {
    module_hom_compose(module_hom_identity(f.dst), f) = Option.some(h) implies h.hom = f.hom
} by {
    if module_hom_compose(module_hom_identity(f.dst), f) = Option.some(h) {
        module_hom_compose_hom(module_hom_identity(f.dst), f, h)
        module_hom_identity_hom(f.dst)
        compose_identity_left(f.hom)
        h.hom = f.hom
    }
}

/// Composing f with the identity module homomorphism on the source is defined as some bundled hom.
theorem module_hom_compose_identity_right_some[R: Ring, M: AddCommGroup, N: AddCommGroup](f: ModuleHom[R, M, N]) {
    exists(h: ModuleHom[R, M, N]) {
        module_hom_compose(f, module_hom_identity(f.src)) = Option.some(h)
    }
} by {
    module_hom_identity_dst(f.src)
    module_hom_compose_some(f, module_hom_identity(f.src))
}

/// Composing f with the identity on the source yields a hom whose underlying function equals f.hom.
theorem module_hom_compose_identity_right_hom[R: Ring, M: AddCommGroup, N: AddCommGroup](f: ModuleHom[R, M, N], h: ModuleHom[R, M, N]) {
    module_hom_compose(f, module_hom_identity(f.src)) = Option.some(h) implies h.hom = f.hom
} by {
    if module_hom_compose(f, module_hom_identity(f.src)) = Option.some(h) {
        module_hom_compose_hom(f, module_hom_identity(f.src), h)
        module_hom_identity_hom(f.src)
        compose_identity_right(f.hom)
        h.hom = f.hom
    }
}

/// Composing the trivial module homomorphism on the destination with f is defined as some bundled hom.
theorem module_hom_compose_trivial_left_some[R: Ring, M: AddCommGroup, N: AddCommGroup, K: AddCommGroup](
    f: ModuleHom[R, M, N], dst: Module[R, K]
) {
    exists(h: ModuleHom[R, M, K]) {
        module_hom_compose(module_hom_trivial(f.dst, dst), f) = Option.some(h)
    }
} by {
    module_hom_trivial_src(f.dst, dst)
    module_hom_compose_some(module_hom_trivial(f.dst, dst), f)
}

/// Composing the trivial hom on the destination with f yields a hom whose underlying function is the trivial linear map.
theorem module_hom_compose_trivial_left_hom[R: Ring, M: AddCommGroup, N: AddCommGroup, K: AddCommGroup](
    f: ModuleHom[R, M, N], dst: Module[R, K], h: ModuleHom[R, M, K]
) {
    module_hom_compose(module_hom_trivial(f.dst, dst), f) = Option.some(h)
        implies h.hom = trivial_linear_map[M, K]
} by {
    if module_hom_compose(module_hom_trivial(f.dst, dst), f) = Option.some(h) {
        module_hom_compose_hom(module_hom_trivial(f.dst, dst), f, h)
        module_hom_trivial_hom(f.dst, dst)
        compose_trivial_linear_map_left[M, N, K](f.hom)
        h.hom = trivial_linear_map[M, K]
    }
}

/// Composing f with the trivial module homomorphism on the source is defined as some bundled hom.
theorem module_hom_compose_trivial_right_some[R: Ring, M: AddCommGroup, N: AddCommGroup, K: AddCommGroup](
    f: ModuleHom[R, N, K], src: Module[R, M]
) {
    exists(h: ModuleHom[R, M, K]) {
        module_hom_compose(f, module_hom_trivial(src, f.src)) = Option.some(h)
    }
} by {
    module_hom_trivial_dst(src, f.src)
    module_hom_compose_some(f, module_hom_trivial(src, f.src))
}

/// Composing f with the trivial hom on the source yields a hom whose underlying function is the trivial linear map.
theorem module_hom_compose_trivial_right_hom[R: Ring, M: AddCommGroup, N: AddCommGroup, K: AddCommGroup](
    f: ModuleHom[R, N, K], src: Module[R, M], h: ModuleHom[R, M, K]
) {
    module_hom_compose(f, module_hom_trivial(src, f.src)) = Option.some(h)
        implies h.hom = trivial_linear_map[M, K]
} by {
    if module_hom_compose(f, module_hom_trivial(src, f.src)) = Option.some(h) {
        module_hom_compose_hom(f, module_hom_trivial(src, f.src), h)
        module_hom_trivial_hom(src, f.src)
        module_hom_is_linear_map(f)
        linear_map_zero(f.src, f.dst, f.hom)
        forall(x: M) {
            f.hom(trivial_linear_map[M, N](x)) = f.hom(N.0)
            compose(f.hom, trivial_linear_map[M, N], x) = trivial_linear_map[M, K](x)
        }
        function_extensionality(compose(f.hom, trivial_linear_map[M, N]), trivial_linear_map[M, K])
        h.hom = trivial_linear_map[M, K]
    }
}

/// Composing the identity on the destination with f evaluates pointwise as f.hom.
theorem module_hom_compose_identity_left_hom_at[R: Ring, M: AddCommGroup, N: AddCommGroup](
    f: ModuleHom[R, M, N], h: ModuleHom[R, M, N], x: M
) {
    module_hom_compose(module_hom_identity(f.dst), f) = Option.some(h)
        implies h.hom(x) = f.hom(x)
} by {
    if module_hom_compose(module_hom_identity(f.dst), f) = Option.some(h) {
        module_hom_compose_identity_left_hom(f, h)
    }
}

/// Composing f with the identity on the source evaluates pointwise as f.hom.
theorem module_hom_compose_identity_right_hom_at[R: Ring, M: AddCommGroup, N: AddCommGroup](
    f: ModuleHom[R, M, N], h: ModuleHom[R, M, N], x: M
) {
    module_hom_compose(f, module_hom_identity(f.src)) = Option.some(h)
        implies h.hom(x) = f.hom(x)
} by {
    if module_hom_compose(f, module_hom_identity(f.src)) = Option.some(h) {
        module_hom_compose_identity_right_hom(f, h)
    }
}

/// Composing the trivial hom on the destination with f evaluates pointwise as zero.
theorem module_hom_compose_trivial_left_hom_at[R: Ring, M: AddCommGroup, N: AddCommGroup, K: AddCommGroup](
    f: ModuleHom[R, M, N], dst: Module[R, K], h: ModuleHom[R, M, K], x: M
) {
    module_hom_compose(module_hom_trivial(f.dst, dst), f) = Option.some(h)
        implies h.hom(x) = K.0
} by {
    if module_hom_compose(module_hom_trivial(f.dst, dst), f) = Option.some(h) {
        module_hom_compose_trivial_left_hom(f, dst, h)
        h.hom(x) = trivial_linear_map[M, K](x)
    }
}

/// Composing f with the trivial hom on the source evaluates pointwise as zero.
theorem module_hom_compose_trivial_right_hom_at[R: Ring, M: AddCommGroup, N: AddCommGroup, K: AddCommGroup](
    f: ModuleHom[R, N, K], src: Module[R, M], h: ModuleHom[R, M, K], x: M
) {
    module_hom_compose(f, module_hom_trivial(src, f.src)) = Option.some(h)
        implies h.hom(x) = K.0
} by {
    if module_hom_compose(f, module_hom_trivial(src, f.src)) = Option.some(h) {
        module_hom_compose_trivial_right_hom(f, src, h)
        h.hom(x) = trivial_linear_map[M, K](x)
    }
}

attributes ModuleHom[R: Ring, M: AddCommGroup, N: AddCommGroup] {
    /// Module hom extensionality from pointwise equality of the underlying function.
    let ext = module_hom_ext[R, M, N]
}
