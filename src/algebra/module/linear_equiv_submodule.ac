from algebra.ring.ring import Ring
from algebra.add_comm_group import AddCommGroup
from algebra.module.module_hom import is_linear_map
from algebra.module.module_iso import LinearEquiv, linear_equiv_to_fn_is_linear_equiv,
    linear_equiv_inv_fn_is_linear_equiv, linear_equiv_inv_fn_is_linear_map,
    linear_equiv_left_inverse
from algebra.module.submodule import Submodule, linear_map_kernel_submodule, linear_map_image_submodule,
    linear_map_preimage_submodule, linear_map_preimage_submodule_contains_eq,
    linear_map_preimage_submodule_mono,
    linear_equiv_kernel_submodule_contains_iff_zero, linear_equiv_image_submodule_contains,
    linear_equiv_kernel_submodule_subset_zero_submodule,
    zero_submodule_subset_linear_equiv_kernel_submodule,
    full_submodule_subset_linear_equiv_image_submodule,
    zero_submodule, zero_submodule_contains_eq, full_submodule, full_submodule_contains_eq,
    submodule_subset, submodule_subset_full, linear_map_image_submodule_carrier_eq,
    linear_map_kernel_submodule_carrier_eq, submodule_subset_antisymm,
    zero_submodule_carrier_eq, full_submodule_carrier_eq

/// The transport of a source submodule across a bundled linear equivalence.
///
/// This is the preimage of the source submodule along the inverse map, so it is
/// a submodule of the equivalence's destination module.
define linear_equiv_submodule_map[R: Ring, M: AddCommGroup, N: AddCommGroup](
    e: LinearEquiv[R, M, N], s: Submodule[R, M]
) -> Submodule[R, N] {
    linear_map_preimage_submodule(e.dst, s, e.inv_fn)
}

/// Membership in the transported submodule is membership of the inverse image
/// in the original submodule, when the original submodule lives over the source.
theorem linear_equiv_submodule_map_contains_eq[R: Ring, M: AddCommGroup, N: AddCommGroup](
    e: LinearEquiv[R, M, N], s: Submodule[R, M], y: N
) {
    s.carrier = e.src implies
        linear_equiv_submodule_map(e, s).contains(y) = s.contains(e.inv_fn(y))
} by {
    if s.carrier = e.src {
        linear_equiv_inv_fn_is_linear_map(e)
        linear_map_preimage_submodule_contains_eq(e.dst, s, e.inv_fn, y)
        linear_equiv_submodule_map(e, s).contains(y) = s.contains(e.inv_fn(y))
    }
}

/// A point of the source belongs to a source submodule exactly when its forward
/// image belongs to the transported submodule.
theorem linear_equiv_submodule_map_contains_forward[R: Ring, M: AddCommGroup, N: AddCommGroup](
    e: LinearEquiv[R, M, N], s: Submodule[R, M], x: M
) {
    s.carrier = e.src implies
        linear_equiv_submodule_map(e, s).contains(e.to_fn(x)) = s.contains(x)
} by {
    if s.carrier = e.src {
        linear_equiv_submodule_map_contains_eq(e, s, e.to_fn(x))
        linear_equiv_left_inverse(e, x)
        linear_equiv_submodule_map(e, s).contains(e.to_fn(x)) = s.contains(x)
    }
}

/// The inverse function of a bundled linear equivalence is linear into any
/// source submodule carrier that is definitionally the equivalence source.
theorem linear_equiv_submodule_inv_fn_is_linear_map[R: Ring, M: AddCommGroup, N: AddCommGroup](
    e: LinearEquiv[R, M, N], s: Submodule[R, M]
) {
    s.carrier = e.src implies is_linear_map(e.dst, s.carrier, e.inv_fn)
} by {
    if s.carrier = e.src {
        linear_equiv_inv_fn_is_linear_map(e)
        is_linear_map(e.dst, s.carrier, e.inv_fn)
    }
}


/// Transport across a bundled linear equivalence is monotone in the source submodule.
theorem linear_equiv_submodule_map_mono[R: Ring, M: AddCommGroup, N: AddCommGroup](
    e: LinearEquiv[R, M, N], s1: Submodule[R, M], s2: Submodule[R, M]
) {
    s1.carrier = e.src and s2.carrier = e.src and submodule_subset(s1, s2) implies
        submodule_subset(linear_equiv_submodule_map(e, s1), linear_equiv_submodule_map(e, s2))
} by {
    if s1.carrier = e.src and s2.carrier = e.src and submodule_subset(s1, s2) {
        linear_equiv_submodule_inv_fn_is_linear_map(e, s1)
        linear_map_preimage_submodule_mono(e.dst, s1, s2, e.inv_fn)
        submodule_subset(linear_map_preimage_submodule(e.dst, s1, e.inv_fn),
            linear_map_preimage_submodule(e.dst, s2, e.inv_fn))
        submodule_subset(linear_equiv_submodule_map(e, s1), linear_equiv_submodule_map(e, s2))
    }
}

/// Under a bundled linear equivalence, the kernel submodule of its forward map
/// contains an element iff that element is the additive identity.
theorem linear_equiv_bundled_kernel_submodule_contains_iff_zero[R: Ring, M: AddCommGroup, N: AddCommGroup](
    e: LinearEquiv[R, M, N], x: M
) {
    linear_map_kernel_submodule(e.src, e.dst, e.to_fn).contains(x) iff x = M.0
} by {
    linear_equiv_to_fn_is_linear_equiv(e)
    linear_equiv_kernel_submodule_contains_iff_zero(e.src, e.dst, e.to_fn, x)
}

/// Under a bundled linear equivalence, the image submodule of its forward map
/// contains every destination element.
theorem linear_equiv_bundled_image_submodule_contains[R: Ring, M: AddCommGroup, N: AddCommGroup](
    e: LinearEquiv[R, M, N], y: N
) {
    linear_map_image_submodule(e.src, e.dst, e.to_fn).contains(y)
} by {
    linear_equiv_to_fn_is_linear_equiv(e)
    linear_equiv_image_submodule_contains(e.src, e.dst, e.to_fn, y)
}

/// Under a bundled linear equivalence, the kernel submodule of its forward map
/// agrees pointwise with the zero submodule of the source.
theorem linear_equiv_bundled_kernel_submodule_contains_eq_zero_submodule[R: Ring, M: AddCommGroup, N: AddCommGroup](
    e: LinearEquiv[R, M, N], x: M
) {
    linear_map_kernel_submodule(e.src, e.dst, e.to_fn).contains(x) =
        zero_submodule(e.src).contains(x)
} by {
    linear_equiv_bundled_kernel_submodule_contains_iff_zero(e, x)
    zero_submodule_contains_eq(e.src, x)
}

/// Under a bundled linear equivalence, the image submodule of its forward map
/// agrees pointwise with the full submodule of the destination.
theorem linear_equiv_bundled_image_submodule_contains_eq_full_submodule[R: Ring, M: AddCommGroup, N: AddCommGroup](
    e: LinearEquiv[R, M, N], y: N
) {
    linear_map_image_submodule(e.src, e.dst, e.to_fn).contains(y) =
        full_submodule(e.dst).contains(y)
} by {
    linear_equiv_bundled_image_submodule_contains(e, y)
    full_submodule_contains_eq(e.dst, y)
}

/// Under a bundled linear equivalence, the kernel submodule of the forward map
/// is contained in the zero submodule of the source.
theorem linear_equiv_bundled_kernel_submodule_subset_zero_submodule[R: Ring, M: AddCommGroup, N: AddCommGroup](
    e: LinearEquiv[R, M, N]
) {
    submodule_subset(
        linear_map_kernel_submodule(e.src, e.dst, e.to_fn),
        zero_submodule(e.src))
} by {
    linear_equiv_to_fn_is_linear_equiv(e)
    linear_equiv_kernel_submodule_subset_zero_submodule(e.src, e.dst, e.to_fn)
}

/// Under a bundled linear equivalence, the zero submodule of the source is
/// contained in the kernel submodule of the forward map.
theorem zero_submodule_subset_linear_equiv_bundled_kernel_submodule[R: Ring, M: AddCommGroup, N: AddCommGroup](
    e: LinearEquiv[R, M, N]
) {
    submodule_subset(
        zero_submodule(e.src),
        linear_map_kernel_submodule(e.src, e.dst, e.to_fn))
} by {
    linear_equiv_to_fn_is_linear_equiv(e)
    zero_submodule_subset_linear_equiv_kernel_submodule(e.src, e.dst, e.to_fn)
}

/// Under a bundled linear equivalence, the image submodule of the forward map
/// is contained in the full submodule of the destination.
theorem linear_equiv_bundled_image_submodule_subset_full_submodule[R: Ring, M: AddCommGroup, N: AddCommGroup](
    e: LinearEquiv[R, M, N]
) {
    submodule_subset(
        linear_map_image_submodule(e.src, e.dst, e.to_fn),
        full_submodule(e.dst))
} by {
    linear_map_image_submodule_carrier_eq(e.src, e.dst, e.to_fn)
    submodule_subset_full(linear_map_image_submodule(e.src, e.dst, e.to_fn))
}

/// Under a bundled linear equivalence, the full submodule of the destination is
/// contained in the image submodule of the forward map.
theorem full_submodule_subset_linear_equiv_bundled_image_submodule[R: Ring, M: AddCommGroup, N: AddCommGroup](
    e: LinearEquiv[R, M, N]
) {
    submodule_subset(
        full_submodule(e.dst),
        linear_map_image_submodule(e.src, e.dst, e.to_fn))
} by {
    linear_equiv_to_fn_is_linear_equiv(e)
    full_submodule_subset_linear_equiv_image_submodule(e.src, e.dst, e.to_fn)
}

/// Under a bundled linear equivalence, the kernel submodule of its inverse map
/// contains an element iff that element is the additive identity.
theorem linear_equiv_bundled_inv_kernel_submodule_contains_iff_zero[R: Ring, M: AddCommGroup, N: AddCommGroup](
    e: LinearEquiv[R, M, N], y: N
) {
    linear_map_kernel_submodule(e.dst, e.src, e.inv_fn).contains(y) iff y = N.0
} by {
    linear_equiv_inv_fn_is_linear_equiv(e)
    linear_equiv_kernel_submodule_contains_iff_zero(e.dst, e.src, e.inv_fn, y)
}

/// Under a bundled linear equivalence, the image submodule of its inverse map
/// contains every source element.
theorem linear_equiv_bundled_inv_image_submodule_contains[R: Ring, M: AddCommGroup, N: AddCommGroup](
    e: LinearEquiv[R, M, N], x: M
) {
    linear_map_image_submodule(e.dst, e.src, e.inv_fn).contains(x)
} by {
    linear_equiv_inv_fn_is_linear_equiv(e)
    linear_equiv_image_submodule_contains(e.dst, e.src, e.inv_fn, x)
}

/// Under a bundled linear equivalence, the kernel submodule of its inverse map
/// agrees pointwise with the zero submodule of the destination.
theorem linear_equiv_bundled_inv_kernel_submodule_contains_eq_zero_submodule[R: Ring, M: AddCommGroup, N: AddCommGroup](
    e: LinearEquiv[R, M, N], y: N
) {
    linear_map_kernel_submodule(e.dst, e.src, e.inv_fn).contains(y) =
        zero_submodule(e.dst).contains(y)
} by {
    linear_equiv_bundled_inv_kernel_submodule_contains_iff_zero(e, y)
    zero_submodule_contains_eq(e.dst, y)
}

/// Under a bundled linear equivalence, the image submodule of its inverse map
/// agrees pointwise with the full submodule of the source.
theorem linear_equiv_bundled_inv_image_submodule_contains_eq_full_submodule[R: Ring, M: AddCommGroup, N: AddCommGroup](
    e: LinearEquiv[R, M, N], x: M
) {
    linear_map_image_submodule(e.dst, e.src, e.inv_fn).contains(x) =
        full_submodule(e.src).contains(x)
} by {
    linear_equiv_bundled_inv_image_submodule_contains(e, x)
    full_submodule_contains_eq(e.src, x)
}

/// Under a bundled linear equivalence, the kernel submodule of the inverse map
/// is contained in the zero submodule of the destination.
theorem linear_equiv_bundled_inv_kernel_submodule_subset_zero_submodule[R: Ring, M: AddCommGroup, N: AddCommGroup](
    e: LinearEquiv[R, M, N]
) {
    submodule_subset(
        linear_map_kernel_submodule(e.dst, e.src, e.inv_fn),
        zero_submodule(e.dst))
} by {
    linear_equiv_inv_fn_is_linear_equiv(e)
    linear_equiv_kernel_submodule_subset_zero_submodule(e.dst, e.src, e.inv_fn)
}

/// Under a bundled linear equivalence, the zero submodule of the destination is
/// contained in the kernel submodule of the inverse map.
theorem zero_submodule_subset_linear_equiv_bundled_inv_kernel_submodule[R: Ring, M: AddCommGroup, N: AddCommGroup](
    e: LinearEquiv[R, M, N]
) {
    submodule_subset(
        zero_submodule(e.dst),
        linear_map_kernel_submodule(e.dst, e.src, e.inv_fn))
} by {
    linear_equiv_inv_fn_is_linear_equiv(e)
    zero_submodule_subset_linear_equiv_kernel_submodule(e.dst, e.src, e.inv_fn)
}

/// Under a bundled linear equivalence, the image submodule of the inverse map
/// is contained in the full submodule of the source.
theorem linear_equiv_bundled_inv_image_submodule_subset_full_submodule[R: Ring, M: AddCommGroup, N: AddCommGroup](
    e: LinearEquiv[R, M, N]
) {
    submodule_subset(
        linear_map_image_submodule(e.dst, e.src, e.inv_fn),
        full_submodule(e.src))
} by {
    linear_map_image_submodule_carrier_eq(e.dst, e.src, e.inv_fn)
    submodule_subset_full(linear_map_image_submodule(e.dst, e.src, e.inv_fn))
}

/// Under a bundled linear equivalence, the full submodule of the source is
/// contained in the image submodule of the inverse map.
theorem full_submodule_subset_linear_equiv_bundled_inv_image_submodule[R: Ring, M: AddCommGroup, N: AddCommGroup](
    e: LinearEquiv[R, M, N]
) {
    submodule_subset(
        full_submodule(e.src),
        linear_map_image_submodule(e.dst, e.src, e.inv_fn))
} by {
    linear_equiv_inv_fn_is_linear_equiv(e)
    full_submodule_subset_linear_equiv_image_submodule(e.dst, e.src, e.inv_fn)
}

/// Under a bundled linear equivalence, the kernel submodule of the forward map
/// equals the zero submodule of the source.
theorem linear_equiv_bundled_kernel_submodule_eq_zero_submodule[R: Ring, M: AddCommGroup, N: AddCommGroup](
    e: LinearEquiv[R, M, N]
) {
    linear_map_kernel_submodule(e.src, e.dst, e.to_fn) = zero_submodule(e.src)
} by {
    linear_map_kernel_submodule_carrier_eq(e.src, e.dst, e.to_fn)
    zero_submodule_carrier_eq(e.src)
    linear_equiv_bundled_kernel_submodule_subset_zero_submodule(e)
    zero_submodule_subset_linear_equiv_bundled_kernel_submodule(e)
    submodule_subset_antisymm(
        linear_map_kernel_submodule(e.src, e.dst, e.to_fn),
        zero_submodule(e.src))
}

/// Under a bundled linear equivalence, the image submodule of the forward map
/// equals the full submodule of the destination.
theorem linear_equiv_bundled_image_submodule_eq_full_submodule[R: Ring, M: AddCommGroup, N: AddCommGroup](
    e: LinearEquiv[R, M, N]
) {
    linear_map_image_submodule(e.src, e.dst, e.to_fn) = full_submodule(e.dst)
} by {
    linear_map_image_submodule_carrier_eq(e.src, e.dst, e.to_fn)
    full_submodule_carrier_eq(e.dst)
    linear_equiv_bundled_image_submodule_subset_full_submodule(e)
    full_submodule_subset_linear_equiv_bundled_image_submodule(e)
    submodule_subset_antisymm(
        linear_map_image_submodule(e.src, e.dst, e.to_fn),
        full_submodule(e.dst))
}

/// Under a bundled linear equivalence, the kernel submodule of the inverse map
/// equals the zero submodule of the destination.
theorem linear_equiv_bundled_inv_kernel_submodule_eq_zero_submodule[R: Ring, M: AddCommGroup, N: AddCommGroup](
    e: LinearEquiv[R, M, N]
) {
    linear_map_kernel_submodule(e.dst, e.src, e.inv_fn) = zero_submodule(e.dst)
} by {
    linear_map_kernel_submodule_carrier_eq(e.dst, e.src, e.inv_fn)
    zero_submodule_carrier_eq(e.dst)
    linear_equiv_bundled_inv_kernel_submodule_subset_zero_submodule(e)
    zero_submodule_subset_linear_equiv_bundled_inv_kernel_submodule(e)
    submodule_subset_antisymm(
        linear_map_kernel_submodule(e.dst, e.src, e.inv_fn),
        zero_submodule(e.dst))
}

/// Under a bundled linear equivalence, the image submodule of the inverse map
/// equals the full submodule of the source.
theorem linear_equiv_bundled_inv_image_submodule_eq_full_submodule[R: Ring, M: AddCommGroup, N: AddCommGroup](
    e: LinearEquiv[R, M, N]
) {
    linear_map_image_submodule(e.dst, e.src, e.inv_fn) = full_submodule(e.src)
} by {
    linear_map_image_submodule_carrier_eq(e.dst, e.src, e.inv_fn)
    full_submodule_carrier_eq(e.src)
    linear_equiv_bundled_inv_image_submodule_subset_full_submodule(e)
    full_submodule_subset_linear_equiv_bundled_inv_image_submodule(e)
    submodule_subset_antisymm(
        linear_map_image_submodule(e.dst, e.src, e.inv_fn),
        full_submodule(e.src))
}
