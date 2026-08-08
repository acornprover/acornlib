/// Additional submodule map/comap API for linear maps.

from algebra.ring.ring import Ring
from algebra.add_comm_group import AddCommGroup
from algebra.module.module import Module
from algebra.module.module_hom import is_linear_map, identity_fn_is_linear_map, linear_map_zero
from data.basic.functions import identity_fn
from data.basic.set import set_image, set_image_monotone
from algebra.module.submodule import Submodule, submodule_subset, submodule_subset_refl,
    submodule_subset_antisymm, submodule_as_set_subset_of_subset,
    submodule_closure, submodule_closure_mono,
    submodule_closure_subset_of_set_subset,
    set_subset_submodule, zero_submodule, zero_submodule_carrier_eq,
    zero_submodule_contains_eq, full_submodule, full_submodule_carrier_eq,
    full_submodule_contains_everything, submodule_contains_zero,
    linear_map_preimage_submodule, linear_map_preimage_submodule_carrier_eq,
    linear_map_preimage_submodule_contains_of_image_in,
    linear_map_preimage_submodule_image_in_of_contains,
    linear_map_preimage_submodule_mono,
    linear_map_preimage_submodule_full_subset,
    full_submodule_subset_linear_map_preimage_full,
    linear_map_preimage_submodule_zero_subset_kernel,
    linear_map_kernel_subset_preimage_submodule_zero,
    linear_map_kernel_submodule, linear_map_kernel_submodule_carrier_eq,
    linear_map_image, linear_map_image_submodule,
    linear_map_image_submodule_carrier_eq,
    linear_map_image_submodule_contains_apply,
    linear_map_image_of_submodule_contains
from algebra.module.submodule_map_comap import submodule_subset_contains_at,
    set_subset_submodule_intro, linear_map_submodule_image,
    linear_map_submodule_image_eq_closure,
    linear_map_submodule_image_carrier_eq,
    linear_map_submodule_image_contains_apply

/// The submodule image construction is monotone in the source submodule.
theorem linear_map_submodule_image_mono[R: Ring, M: AddCommGroup, N: AddCommGroup](
    src: Module[R, M], dst: Module[R, N], f: M -> N,
    s1: Submodule[R, M], s2: Submodule[R, M]
) {
    submodule_subset(s1, s2) implies submodule_subset(
        linear_map_submodule_image(src, dst, f, s1),
        linear_map_submodule_image(src, dst, f, s2))
} by {
    if submodule_subset(s1, s2) {
        submodule_as_set_subset_of_subset(s1, s2)
        s1.as_set.subset(s2.as_set)
        set_image_monotone(s1.as_set, s2.as_set, f)
        set_image(s1.as_set, f).subset(set_image(s2.as_set, f))
        submodule_closure_mono(dst, set_image(s1.as_set, f), set_image(s2.as_set, f))
        submodule_subset(
            submodule_closure(dst, set_image(s1.as_set, f)),
            submodule_closure(dst, set_image(s2.as_set, f)))
        linear_map_submodule_image_eq_closure(src, dst, f, s1)
        linear_map_submodule_image_eq_closure(src, dst, f, s2)
        submodule_subset(
            linear_map_submodule_image(src, dst, f, s1),
            linear_map_submodule_image(src, dst, f, s2))
    }
}

/// A pointwise maps-into condition is enough to bound the submodule image.
theorem linear_map_submodule_image_subset_of_maps_into[R: Ring, M: AddCommGroup, N: AddCommGroup](
    src: Module[R, M], dst: Module[R, N], f: M -> N,
    s: Submodule[R, M], t: Submodule[R, N]
) {
    t.carrier = dst and (forall(x: M) { s.contains(x) implies t.contains(f(x)) })
    implies submodule_subset(linear_map_submodule_image(src, dst, f, s), t)
} by {
    if t.carrier = dst and forall(x: M) { s.contains(x) implies t.contains(f(x)) } {
        forall(y: N) {
            if set_image(s.as_set, f).contains(y) {
                let x: M satisfy {
                    s.as_set.contains(x) and y = f(x)
                }
                s.contains(x)
                t.contains(f(x))
                t.contains(y)
            }
        }
        set_subset_submodule_intro(set_image(s.as_set, f), t)
        set_subset_submodule(set_image(s.as_set, f), t)
        submodule_closure_subset_of_set_subset(dst, set_image(s.as_set, f), t)
        submodule_subset(submodule_closure(dst, set_image(s.as_set, f)), t)
        linear_map_submodule_image_eq_closure(src, dst, f, s)
        submodule_subset(linear_map_submodule_image(src, dst, f, s), t)
    }
}

/// Bounding the submodule image is equivalent to checking the images of source members.
theorem linear_map_submodule_image_subset_iff_maps_into[R: Ring, M: AddCommGroup, N: AddCommGroup](
    src: Module[R, M], dst: Module[R, N], f: M -> N,
    s: Submodule[R, M], t: Submodule[R, N]
) {
    t.carrier = dst implies
        (submodule_subset(linear_map_submodule_image(src, dst, f, s), t) =
            forall(x: M) { s.contains(x) implies t.contains(f(x)) })
} by {
    if t.carrier = dst {
        if submodule_subset(linear_map_submodule_image(src, dst, f, s), t) {
            forall(x: M) {
                if s.contains(x) {
                    linear_map_submodule_image_contains_apply(src, dst, f, s, x)
                    linear_map_submodule_image(src, dst, f, s).contains(f(x))
                    submodule_subset_contains_at(linear_map_submodule_image(src, dst, f, s), t, f(x))
                    t.contains(f(x))
                }
            }
        }
        if forall(x: M) { s.contains(x) implies t.contains(f(x)) } {
            linear_map_submodule_image_subset_of_maps_into(src, dst, f, s, t)
            submodule_subset(linear_map_submodule_image(src, dst, f, s), t)
        }
        submodule_subset(linear_map_submodule_image(src, dst, f, s), t) =
            forall(x: M) { s.contains(x) implies t.contains(f(x)) }
    }
}

/// A carrier-normalized monotonicity form for preimage submodules.
theorem linear_map_preimage_submodule_mono_of_subset[R: Ring, M: AddCommGroup, N: AddCommGroup](
    src: Module[R, M], dst: Module[R, N], f: M -> N,
    t1: Submodule[R, N], t2: Submodule[R, N]
) {
    t1.carrier = dst and t2.carrier = dst and is_linear_map(src, dst, f) and submodule_subset(t1, t2)
    implies submodule_subset(
        linear_map_preimage_submodule(src, t1, f),
        linear_map_preimage_submodule(src, t2, f))
} by {
    if t1.carrier = dst and t2.carrier = dst and is_linear_map(src, dst, f) and submodule_subset(t1, t2) {
        t1.carrier = t2.carrier
        is_linear_map(src, t1.carrier, f)
        linear_map_preimage_submodule_mono(src, t1, t2, f)
        submodule_subset(
            linear_map_preimage_submodule(src, t1, f),
            linear_map_preimage_submodule(src, t2, f))
    }
}

/// The unit inequality of the map/comap adjunction: `s <= comap (map s)`.
theorem linear_map_submodule_subset_preimage_image[R: Ring, M: AddCommGroup, N: AddCommGroup](
    src: Module[R, M], dst: Module[R, N], f: M -> N, s: Submodule[R, M]
) {
    is_linear_map(src, dst, f) and s.carrier = src implies submodule_subset(
        s,
        linear_map_preimage_submodule(src, linear_map_submodule_image(src, dst, f, s), f))
} by {
    let image = linear_map_submodule_image(src, dst, f, s)
    if is_linear_map(src, dst, f) and s.carrier = src {
        linear_map_submodule_image_carrier_eq(src, dst, f, s)
        image.carrier = dst
        is_linear_map(src, image.carrier, f)
        forall(x: M) {
            if s.contains(x) {
                linear_map_submodule_image_contains_apply(src, dst, f, s, x)
                image.contains(f(x))
                linear_map_preimage_submodule_contains_of_image_in(src, image, f, x)
                linear_map_preimage_submodule(src, image, f).contains(x)
            }
        }
    }
}

/// The counit inequality of the map/comap adjunction: `map (comap t) <= t`.
theorem linear_map_submodule_image_preimage_subset[R: Ring, M: AddCommGroup, N: AddCommGroup](
    src: Module[R, M], dst: Module[R, N], f: M -> N, t: Submodule[R, N]
) {
    is_linear_map(src, dst, f) and t.carrier = dst implies submodule_subset(
        linear_map_submodule_image(src, dst, f, linear_map_preimage_submodule(src, t, f)),
        t)
} by {
    let preimage = linear_map_preimage_submodule(src, t, f)
    if is_linear_map(src, dst, f) and t.carrier = dst {
        is_linear_map(src, t.carrier, f)
        forall(x: M) {
            if preimage.contains(x) {
                linear_map_preimage_submodule_image_in_of_contains(src, t, f, x)
                t.contains(f(x))
            }
        }
        linear_map_submodule_image_subset_of_maps_into(src, dst, f, preimage, t)
        submodule_subset(linear_map_submodule_image(src, dst, f, preimage), t)
    }
}

/// The preimage of the full submodule is the full source submodule.
theorem linear_map_preimage_submodule_full_eq_full[R: Ring, M: AddCommGroup, N: AddCommGroup](
    src: Module[R, M], dst: Module[R, N], f: M -> N
) {
    is_linear_map(src, dst, f) implies
        linear_map_preimage_submodule(src, full_submodule(dst), f) = full_submodule(src)
} by {
    if is_linear_map(src, dst, f) {
        linear_map_preimage_submodule_full_subset(src, dst, f)
        full_submodule_subset_linear_map_preimage_full(src, dst, f)
        linear_map_preimage_submodule_carrier_eq(src, full_submodule(dst), f)
        full_submodule_carrier_eq(src)
        linear_map_preimage_submodule(src, full_submodule(dst), f).carrier = src
        full_submodule(src).carrier = src
        linear_map_preimage_submodule(src, full_submodule(dst), f).carrier = full_submodule(src).carrier
        submodule_subset_antisymm(linear_map_preimage_submodule(src, full_submodule(dst), f), full_submodule(src))
        linear_map_preimage_submodule(src, full_submodule(dst), f) = full_submodule(src)
    }
}

/// The preimage of the zero submodule is the kernel submodule.
theorem linear_map_preimage_submodule_zero_eq_kernel[R: Ring, M: AddCommGroup, N: AddCommGroup](
    src: Module[R, M], dst: Module[R, N], f: M -> N
) {
    is_linear_map(src, dst, f) implies
        linear_map_preimage_submodule(src, zero_submodule(dst), f) = linear_map_kernel_submodule(src, dst, f)
} by {
    if is_linear_map(src, dst, f) {
        linear_map_preimage_submodule_zero_subset_kernel(src, dst, f)
        linear_map_kernel_subset_preimage_submodule_zero(src, dst, f)
        linear_map_preimage_submodule_carrier_eq(src, zero_submodule(dst), f)
        linear_map_kernel_submodule_carrier_eq(src, dst, f)
        linear_map_preimage_submodule(src, zero_submodule(dst), f).carrier = src
        linear_map_kernel_submodule(src, dst, f).carrier = src
        linear_map_preimage_submodule(src, zero_submodule(dst), f).carrier =
            linear_map_kernel_submodule(src, dst, f).carrier
        submodule_subset_antisymm(
            linear_map_preimage_submodule(src, zero_submodule(dst), f),
            linear_map_kernel_submodule(src, dst, f))
        linear_map_preimage_submodule(src, zero_submodule(dst), f) = linear_map_kernel_submodule(src, dst, f)
    }
}

/// Preimage under the identity linear map is the original submodule.
theorem linear_map_preimage_submodule_identity[R: Ring, M: AddCommGroup](
    src: Module[R, M], t: Submodule[R, M]
) {
    t.carrier = src implies linear_map_preimage_submodule(src, t, identity_fn[M]) = t
} by {
    let preimage = linear_map_preimage_submodule(src, t, identity_fn[M])
    if t.carrier = src {
        identity_fn_is_linear_map(src)
        is_linear_map(src, t.carrier, identity_fn[M])
        linear_map_preimage_submodule_carrier_eq(src, t, identity_fn[M])
        preimage.carrier = src
        preimage.carrier = t.carrier
        forall(x: M) {
            if preimage.contains(x) {
                linear_map_preimage_submodule_image_in_of_contains(src, t, identity_fn[M], x)
                t.contains(identity_fn[M](x))
                identity_fn[M](x) = x
                t.contains(x)
            }
        }
        submodule_subset(preimage, t)
        forall(x: M) {
            if t.contains(x) {
                identity_fn[M](x) = x
                t.contains(identity_fn[M](x))
                linear_map_preimage_submodule_contains_of_image_in(src, t, identity_fn[M], x)
                preimage.contains(x)
            }
        }
        submodule_subset(t, preimage)
        preimage.carrier = t.carrier
        submodule_subset_antisymm(preimage, t)
        preimage = t
    }
}

/// A restricted submodule image is contained in the global image submodule.
theorem linear_map_submodule_image_subset_image_submodule[R: Ring, M: AddCommGroup, N: AddCommGroup](
    src: Module[R, M], dst: Module[R, N], f: M -> N, s: Submodule[R, M]
) {
    is_linear_map(src, dst, f) implies submodule_subset(
        linear_map_submodule_image(src, dst, f, s),
        linear_map_image_submodule(src, dst, f))
} by {
    if is_linear_map(src, dst, f) {
        linear_map_image_submodule_carrier_eq(src, dst, f)
        forall(x: M) {
            if s.contains(x) {
                linear_map_image_submodule_contains_apply(src, dst, f, x)
                linear_map_image_submodule(src, dst, f).contains(f(x))
            }
        }
        linear_map_submodule_image_subset_of_maps_into(src, dst, f, s, linear_map_image_submodule(src, dst, f))
        submodule_subset(linear_map_submodule_image(src, dst, f, s), linear_map_image_submodule(src, dst, f))
    }
}

/// The global image submodule is contained in the image of the full source submodule.
theorem linear_map_image_submodule_subset_submodule_image_full[R: Ring, M: AddCommGroup, N: AddCommGroup](
    src: Module[R, M], dst: Module[R, N], f: M -> N
) {
    is_linear_map(src, dst, f) implies submodule_subset(
        linear_map_image_submodule(src, dst, f),
        linear_map_submodule_image(src, dst, f, full_submodule(src)))
} by {
    if is_linear_map(src, dst, f) {
        forall(y: N) {
            if linear_map_image_submodule(src, dst, f).contains(y) {
                linear_map_image_of_submodule_contains(src, dst, f, y)
                linear_map_image[M, N](f, y)
                let x: M satisfy {
                    f(x) = y
                }
                full_submodule_contains_everything(src, x)
                linear_map_submodule_image_contains_apply(src, dst, f, full_submodule(src), x)
                linear_map_submodule_image(src, dst, f, full_submodule(src)).contains(f(x))
                linear_map_submodule_image(src, dst, f, full_submodule(src)).contains(y)
            }
        }
    }
}

/// The image of the full source submodule is the usual image submodule.
theorem linear_map_submodule_image_full_eq_image_submodule[R: Ring, M: AddCommGroup, N: AddCommGroup](
    src: Module[R, M], dst: Module[R, N], f: M -> N
) {
    is_linear_map(src, dst, f) implies
        linear_map_submodule_image(src, dst, f, full_submodule(src)) = linear_map_image_submodule(src, dst, f)
} by {
    if is_linear_map(src, dst, f) {
        linear_map_submodule_image_subset_image_submodule(src, dst, f, full_submodule(src))
        linear_map_image_submodule_subset_submodule_image_full(src, dst, f)
        linear_map_submodule_image_carrier_eq(src, dst, f, full_submodule(src))
        linear_map_image_submodule_carrier_eq(src, dst, f)
        linear_map_submodule_image(src, dst, f, full_submodule(src)).carrier = dst
        linear_map_image_submodule(src, dst, f).carrier = dst
        linear_map_submodule_image(src, dst, f, full_submodule(src)).carrier =
            linear_map_image_submodule(src, dst, f).carrier
        submodule_subset_antisymm(
            linear_map_submodule_image(src, dst, f, full_submodule(src)),
            linear_map_image_submodule(src, dst, f))
        linear_map_submodule_image(src, dst, f, full_submodule(src)) = linear_map_image_submodule(src, dst, f)
    }
}

/// The image of the zero source submodule under a linear map is the zero target submodule.
theorem linear_map_submodule_image_zero_eq_zero[R: Ring, M: AddCommGroup, N: AddCommGroup](
    src: Module[R, M], dst: Module[R, N], f: M -> N
) {
    is_linear_map(src, dst, f) implies
        linear_map_submodule_image(src, dst, f, zero_submodule(src)) = zero_submodule(dst)
} by {
    let zsrc = zero_submodule(src)
    let zdst = zero_submodule(dst)
    let image = linear_map_submodule_image(src, dst, f, zsrc)
    if is_linear_map(src, dst, f) {
        zero_submodule_carrier_eq(dst)
        zdst.carrier = dst
        forall(x: M) {
            if zsrc.contains(x) {
                zero_submodule_contains_eq(src, x)
                x = M.0
                linear_map_zero(src, dst, f)
                f(M.0) = N.0
                f(x) = N.0
                zero_submodule_contains_eq(dst, f(x))
                zdst.contains(f(x))
            }
        }
        linear_map_submodule_image_subset_of_maps_into(src, dst, f, zsrc, zdst)
        submodule_subset(linear_map_submodule_image(src, dst, f, zsrc), zdst)
        submodule_subset(image, zdst)
        submodule_contains_zero(zsrc)
        zsrc.contains(M.0)
        linear_map_submodule_image_contains_apply(src, dst, f, zsrc, M.0)
        image.contains(f(M.0))
        linear_map_zero(src, dst, f)
        f(M.0) = N.0
        image.contains(N.0)
        forall(y: N) {
            if zdst.contains(y) {
                zero_submodule_contains_eq(dst, y)
                y = N.0
                image.contains(y)
            }
        }
        submodule_subset(zdst, image)
        linear_map_submodule_image_carrier_eq(src, dst, f, zsrc)
        image.carrier = dst
        zdst.carrier = dst
        image.carrier = zdst.carrier
        submodule_subset_antisymm(image, zdst)
        image = zdst
    }
}
