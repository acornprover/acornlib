/// Galois connections between partially ordered types.

from order import PartialOrder, lte_refl, lte_trans, lte_antisymm
from data.basic.functions import is_surjective_fn, identity_fn, compose
from order import is_monotone, is_order_embedding, is_order_surjection, monotone_step,
    order_embedding_reflects_lte
from lattice import MeetSemilattice, JoinSemilattice, meet_lte_left, meet_lte_right,
    lte_meet_of_bounds, lte_join_left, lte_join_right, join_lte_of_bounds
from order_bounds import is_lower_bound, is_upper_bound, is_lub, is_glb,
    is_lub_is_upper_bound, is_lub_le_upper_bound, is_glb_is_lower_bound,
    is_glb_ge_lower_bound, join_is_lub, meet_is_glb, is_in_pair,
    is_in_pair_iff, is_lub_unique, is_glb_unique, is_lub_of_pred_eq,
    is_glb_of_pred_eq

/// True if `lower` is left adjoint to `upper`.
define is_galois_connection[A: PartialOrder, B: PartialOrder](
    lower: A -> B,
    upper: B -> A
) -> Bool {
    forall(a: A, b: B) {
        lower(a) <= b = (a <= upper(b))
    }
}

/// A Galois connection is exactly its order-adjunction law.
theorem galois_connection_at[A: PartialOrder, B: PartialOrder](
    lower: A -> B,
    upper: B -> A,
    a: A,
    b: B
) {
    is_galois_connection(lower, upper) implies
    (lower(a) <= b = (a <= upper(b)))
} by {
    if is_galois_connection(lower, upper) {
        is_galois_connection(lower, upper) = forall(x: A, y: B) {
            lower(x) <= y = (x <= upper(y))
        }
        lower(a) <= b = (a <= upper(b))
    }
}

/// The left-to-right direction of a Galois connection.
theorem galois_connection_le_upper[A: PartialOrder, B: PartialOrder](
    lower: A -> B,
    upper: B -> A,
    a: A,
    b: B
) {
    is_galois_connection(lower, upper) and lower(a) <= b implies a <= upper(b)
} by {
    if is_galois_connection(lower, upper) and lower(a) <= b {
        galois_connection_at(lower, upper, a, b)
        a <= upper(b)
    }
}

/// The right-to-left direction of a Galois connection.
theorem galois_connection_lower_le[A: PartialOrder, B: PartialOrder](
    lower: A -> B,
    upper: B -> A,
    a: A,
    b: B
) {
    is_galois_connection(lower, upper) and a <= upper(b) implies lower(a) <= b
} by {
    if is_galois_connection(lower, upper) and a <= upper(b) {
        galois_connection_at(lower, upper, a, b)
        lower(a) <= b
    }
}

/// The lower adjoint is monotone.
theorem galois_connection_lower_monotone[A: PartialOrder, B: PartialOrder](
    lower: A -> B,
    upper: B -> A
) {
    is_galois_connection(lower, upper) implies is_monotone(lower)
} by {
    if is_galois_connection(lower, upper) {
        forall(a: A, b: A) {
            if a <= b {
                lte_refl(lower(b))
                galois_connection_le_upper(lower, upper, b, lower(b))
                lte_trans(a, b, upper(lower(b)))
                galois_connection_lower_le(lower, upper, a, lower(b))
                lower(a) <= lower(b)
            }
        }
        is_monotone(lower)
    }
}

/// The upper adjoint is monotone.
theorem galois_connection_upper_monotone[A: PartialOrder, B: PartialOrder](
    lower: A -> B,
    upper: B -> A
) {
    is_galois_connection(lower, upper) implies is_monotone(upper)
} by {
    if is_galois_connection(lower, upper) {
        forall(a: B, b: B) {
            if a <= b {
                lte_refl(upper(a))
                galois_connection_lower_le(lower, upper, upper(a), a)
                lte_trans(lower(upper(a)), a, b)
                galois_connection_le_upper(lower, upper, upper(a), b)
                upper(a) <= upper(b)
            }
        }
        is_monotone(upper)
    }
}

/// The predicate image of `p` under a function.
define image_pred[A, B](f: A -> B, p: A -> Bool, y: B) -> Bool {
    exists(x: A) { p(x) and f(x) = y }
}

/// A lower adjoint carries a supremum to the supremum of the predicate image.
theorem galois_connection_lower_preserves_lub[A: PartialOrder, B: PartialOrder](
    lower: A -> B,
    upper: B -> A,
    p: A -> Bool,
    a: A
) {
    is_galois_connection(lower, upper) and is_lub(p, a) implies
    is_lub(image_pred(lower, p), lower(a))
} by {
    if is_galois_connection(lower, upper) and is_lub(p, a) {
        is_lub_is_upper_bound(p, a)
        forall(y: B) {
            if image_pred(lower, p, y) {
                let x: A satisfy { p(x) and lower(x) = y }
                is_upper_bound(p, a) = forall(z: A) {
                    p(z) implies z <= a
                }
                lte_refl(lower(a))
                galois_connection_le_upper(lower, upper, a, lower(a))
                lte_trans(x, a, upper(lower(a)))
                galois_connection_lower_le(lower, upper, x, lower(a))
                lower(x) <= lower(a)
                y <= lower(a)
            }
        }
        is_upper_bound(image_pred(lower, p), lower(a))
        forall(c: B) {
            if is_upper_bound(image_pred(lower, p), c) {
                forall(x: A) {
                    if p(x) {
                        image_pred(lower, p, lower(x))
                        is_upper_bound(image_pred(lower, p), c) = forall(y: B) {
                            image_pred(lower, p, y) implies y <= c
                        }
                        galois_connection_le_upper(lower, upper, x, c)
                        x <= upper(c)
                    }
                }
                is_upper_bound(p, upper(c))
                is_lub_le_upper_bound(p, a, upper(c))
                galois_connection_lower_le(lower, upper, a, c)
                lower(a) <= c
            }
        }
        is_lub(image_pred(lower, p), lower(a))
    }
}

/// An upper adjoint carries an infimum to the infimum of the predicate image.
theorem galois_connection_upper_preserves_glb[A: PartialOrder, B: PartialOrder](
    lower: A -> B,
    upper: B -> A,
    q: B -> Bool,
    b: B
) {
    is_galois_connection(lower, upper) and is_glb(q, b) implies
    is_glb(image_pred(upper, q), upper(b))
} by {
    if is_galois_connection(lower, upper) and is_glb(q, b) {
        is_glb_is_lower_bound(q, b)
        forall(y: A) {
            if image_pred(upper, q, y) {
                let x: B satisfy { q(x) and upper(x) = y }
                is_lower_bound(q, b) = forall(z: B) {
                    q(z) implies b <= z
                }
                lte_refl(upper(b))
                galois_connection_lower_le(lower, upper, upper(b), b)
                lte_trans(lower(upper(b)), b, x)
                galois_connection_le_upper(lower, upper, upper(b), x)
                upper(b) <= upper(x)
                upper(b) <= y
            }
        }
        is_lower_bound(image_pred(upper, q), upper(b))
        forall(c: A) {
            if is_lower_bound(image_pred(upper, q), c) {
                forall(x: B) {
                    if q(x) {
                        image_pred(upper, q, upper(x))
                        is_lower_bound(image_pred(upper, q), c) = forall(y: A) {
                            image_pred(upper, q, y) implies c <= y
                        }
                        galois_connection_lower_le(lower, upper, c, x)
                        lower(c) <= x
                    }
                }
                is_lower_bound(q, lower(c))
                is_glb_ge_lower_bound(q, b, lower(c))
                galois_connection_le_upper(lower, upper, c, b)
                c <= upper(b)
            }
        }
        is_glb(image_pred(upper, q), upper(b))
    }
}

/// A lower adjoint preserves binary joins.
theorem galois_connection_lower_preserves_join[A: JoinSemilattice, B: JoinSemilattice](
    lower: A -> B,
    upper: B -> A,
    a: A,
    b: A
) {
    is_galois_connection(lower, upper) implies lower(a.join(b)) = lower(a).join(lower(b))
} by {
    if is_galois_connection(lower, upper) {
        let p: A -> Bool = is_in_pair(a, b)
        let q: B -> Bool = is_in_pair(lower(a), lower(b))
        join_is_lub(a, b)
        galois_connection_lower_preserves_lub(lower, upper, p, a.join(b))
        is_lub(image_pred(lower, p), lower(a.join(b)))
        forall(y: B) {
            if image_pred(lower, p, y) {
                let x: A satisfy { p(x) and lower(x) = y }
                is_in_pair_iff(a, b, x)
                if x = a {
                    q(y)
                } else {
                    x = b
                    q(y)
                }
            }
            if q(y) {
                is_in_pair_iff(lower(a), lower(b), y)
                if y = lower(a) {
                    image_pred(lower, p, y)
                } else {
                    p(b)
                    image_pred(lower, p, y)
                }
            }
            image_pred(lower, p, y) iff q(y)
        }
        forall(y: B) { image_pred(lower, p, y) iff q(y) }
        is_lub_of_pred_eq(image_pred(lower, p), q, lower(a.join(b)))
        join_is_lub(lower(a), lower(b))
        is_lub(q, lower(a).join(lower(b)))
        is_lub_unique(q, lower(a.join(b)), lower(a).join(lower(b)))
        lower(a.join(b)) = lower(a).join(lower(b))
    }
}

/// An upper adjoint preserves binary meets.
theorem galois_connection_upper_preserves_meet[A: MeetSemilattice, B: MeetSemilattice](
    lower: A -> B,
    upper: B -> A,
    c: B,
    d: B
) {
    is_galois_connection(lower, upper) implies upper(c.meet(d)) = upper(c).meet(upper(d))
} by {
    if is_galois_connection(lower, upper) {
        let q: B -> Bool = is_in_pair(c, d)
        let p: A -> Bool = is_in_pair(upper(c), upper(d))
        meet_is_glb(c, d)
        galois_connection_upper_preserves_glb(lower, upper, q, c.meet(d))
        is_glb(image_pred(upper, q), upper(c.meet(d)))
        forall(y: A) {
            if image_pred(upper, q, y) {
                let x: B satisfy { q(x) and upper(x) = y }
                is_in_pair_iff(c, d, x)
                if x = c {
                    p(y)
                } else {
                    x = d
                    p(y)
                }
            }
            if p(y) {
                is_in_pair_iff(upper(c), upper(d), y)
                if y = upper(c) {
                    image_pred(upper, q, y)
                } else {
                    q(d)
                    image_pred(upper, q, y)
                }
            }
            image_pred(upper, q, y) iff p(y)
        }
        forall(y: A) { image_pred(upper, q, y) iff p(y) }
        is_glb_of_pred_eq(image_pred(upper, q), p, upper(c.meet(d)))
        meet_is_glb(upper(c), upper(d))
        is_glb(p, upper(c).meet(upper(d)))
        is_glb_unique(p, upper(c.meet(d)), upper(c).meet(upper(d)))
        upper(c.meet(d)) = upper(c).meet(upper(d))
    }
}

/// The unit inequality of a Galois connection.
theorem galois_connection_unit[A: PartialOrder, B: PartialOrder](
    lower: A -> B,
    upper: B -> A,
    a: A
) {
    is_galois_connection(lower, upper) implies a <= upper(lower(a))
} by {
    if is_galois_connection(lower, upper) {
        lte_refl(lower(a))
        galois_connection_le_upper(lower, upper, a, lower(a))
        a <= upper(lower(a))
    }
}

/// The counit inequality of a Galois connection.
theorem galois_connection_counit[A: PartialOrder, B: PartialOrder](
    lower: A -> B,
    upper: B -> A,
    b: B
) {
    is_galois_connection(lower, upper) implies lower(upper(b)) <= b
} by {
    if is_galois_connection(lower, upper) {
        lte_refl(upper(b))
        galois_connection_lower_le(lower, upper, upper(b), b)
        lower(upper(b)) <= b
    }
}

/// The closure operator on the lower side induced by a Galois connection.
define galois_closure[A: PartialOrder, B: PartialOrder](
    lower: A -> B,
    upper: B -> A,
    a: A
) -> A {
    upper(lower(a))
}

/// The kernel operator on the upper side induced by a Galois connection.
define galois_kernel[A: PartialOrder, B: PartialOrder](
    lower: A -> B,
    upper: B -> A,
    b: B
) -> B {
    lower(upper(b))
}

/// The Galois closure is the upper adjoint after the lower adjoint.
theorem galois_closure_at[A: PartialOrder, B: PartialOrder](
    lower: A -> B,
    upper: B -> A,
    a: A
) {
    galois_closure(lower, upper, a) = upper(lower(a))
}

/// The Galois kernel is the lower adjoint after the upper adjoint.
theorem galois_kernel_at[A: PartialOrder, B: PartialOrder](
    lower: A -> B,
    upper: B -> A,
    b: B
) {
    galois_kernel(lower, upper, b) = lower(upper(b))
}

/// The Galois closure is extensive.
theorem galois_closure_extensive[A: PartialOrder, B: PartialOrder](
    lower: A -> B,
    upper: B -> A,
    a: A
) {
    is_galois_connection(lower, upper) implies a <= galois_closure(lower, upper, a)
} by {
    if is_galois_connection(lower, upper) {
        galois_connection_unit(lower, upper, a)
        galois_closure_at(lower, upper, a)
        a <= galois_closure(lower, upper, a)
    }
}

/// The Galois kernel is reductive.
theorem galois_kernel_reductive[A: PartialOrder, B: PartialOrder](
    lower: A -> B,
    upper: B -> A,
    b: B
) {
    is_galois_connection(lower, upper) implies galois_kernel(lower, upper, b) <= b
} by {
    if is_galois_connection(lower, upper) {
        galois_connection_counit(lower, upper, b)
        galois_kernel_at(lower, upper, b)
        galois_kernel(lower, upper, b) <= b
    }
}

/// The Galois closure is monotone.
theorem galois_closure_monotone[A: PartialOrder, B: PartialOrder](
    lower: A -> B,
    upper: B -> A
) {
    is_galois_connection(lower, upper) implies is_monotone(galois_closure(lower, upper))
} by {
    if is_galois_connection(lower, upper) {
        galois_connection_lower_monotone(lower, upper)
        galois_connection_upper_monotone(lower, upper)
        forall(a: A, b: A) {
            if a <= b {
                monotone_step(lower, a, b)
                monotone_step(upper, lower(a), lower(b))
                upper(lower(a)) <= upper(lower(b))
                galois_closure_at(lower, upper, a)
                galois_closure_at(lower, upper, b)
                galois_closure(lower, upper, a) <= galois_closure(lower, upper, b)
            }
        }
        is_monotone(galois_closure(lower, upper))
    }
}

/// The Galois kernel is monotone.
theorem galois_kernel_monotone[A: PartialOrder, B: PartialOrder](
    lower: A -> B,
    upper: B -> A
) {
    is_galois_connection(lower, upper) implies is_monotone(galois_kernel(lower, upper))
} by {
    if is_galois_connection(lower, upper) {
        galois_connection_lower_monotone(lower, upper)
        galois_connection_upper_monotone(lower, upper)
        forall(a: B, b: B) {
            if a <= b {
                monotone_step(upper, a, b)
                monotone_step(lower, upper(a), upper(b))
                lower(upper(a)) <= lower(upper(b))
                galois_kernel_at(lower, upper, a)
                galois_kernel_at(lower, upper, b)
                galois_kernel(lower, upper, a) <= galois_kernel(lower, upper, b)
            }
        }
        is_monotone(galois_kernel(lower, upper))
    }
}

/// The closure composite of a Galois connection is idempotent.
theorem galois_closure_raw_idempotent[A: PartialOrder, B: PartialOrder](
    lower: A -> B,
    upper: B -> A,
    a: A
) {
    is_galois_connection(lower, upper) implies upper(lower(upper(lower(a)))) = upper(lower(a))
} by {
    if is_galois_connection(lower, upper) {
        let c = upper(lower(a))
        galois_connection_unit(lower, upper, c)

        galois_connection_counit(lower, upper, lower(a))
        galois_connection_upper_monotone(lower, upper)
        monotone_step(upper, lower(c), lower(a))
        upper(lower(c)) <= upper(lower(a))
        upper(lower(c)) <= c
        lte_antisymm(upper(lower(c)), c)
        upper(lower(c)) = c
        upper(lower(upper(lower(a)))) = upper(lower(a))
    }
}

/// The Galois closure is idempotent.
theorem galois_closure_idempotent[A: PartialOrder, B: PartialOrder](
    lower: A -> B,
    upper: B -> A,
    a: A
) {
    is_galois_connection(lower, upper) implies
    galois_closure(lower, upper, galois_closure(lower, upper, a)) =
    galois_closure(lower, upper, a)
} by {
    if is_galois_connection(lower, upper) {
        galois_closure_at(lower, upper, a)
        galois_closure_at(lower, upper, galois_closure(lower, upper, a))
        galois_closure_raw_idempotent(lower, upper, a)
        upper(lower(upper(lower(a)))) = upper(lower(a))
        galois_closure(lower, upper, galois_closure(lower, upper, a)) =
        galois_closure(lower, upper, a)
    }
}

/// The kernel composite of a Galois connection is idempotent.
theorem galois_kernel_raw_idempotent[A: PartialOrder, B: PartialOrder](
    lower: A -> B,
    upper: B -> A,
    b: B
) {
    is_galois_connection(lower, upper) implies lower(upper(lower(upper(b)))) = lower(upper(b))
} by {
    if is_galois_connection(lower, upper) {
        let c = lower(upper(b))
        galois_connection_counit(lower, upper, c)

        galois_connection_unit(lower, upper, upper(b))
        galois_connection_lower_monotone(lower, upper)
        monotone_step(lower, upper(b), upper(c))
        lower(upper(b)) <= lower(upper(c))
        c <= lower(upper(c))
        lte_antisymm(lower(upper(c)), c)
        lower(upper(c)) = c
        lower(upper(lower(upper(b)))) = lower(upper(b))
    }
}

/// The Galois kernel is idempotent.
theorem galois_kernel_idempotent[A: PartialOrder, B: PartialOrder](
    lower: A -> B,
    upper: B -> A,
    b: B
) {
    is_galois_connection(lower, upper) implies
    galois_kernel(lower, upper, galois_kernel(lower, upper, b)) =
    galois_kernel(lower, upper, b)
} by {
    if is_galois_connection(lower, upper) {
        galois_kernel_at(lower, upper, b)
        galois_kernel_at(lower, upper, galois_kernel(lower, upper, b))
        galois_kernel_raw_idempotent(lower, upper, b)
        lower(upper(lower(upper(b)))) = lower(upper(b))
        galois_kernel(lower, upper, galois_kernel(lower, upper, b)) =
        galois_kernel(lower, upper, b)
    }
}

/// True if a Galois connection has an exact upper-side counit.
define is_galois_insertion[A: PartialOrder, B: PartialOrder](
    lower: A -> B,
    upper: B -> A
) -> Bool {
    is_galois_connection(lower, upper) and forall(b: B) {
        lower(upper(b)) = b
    }
}

/// True if a Galois connection has an exact lower-side unit.
define is_galois_coinsertion[A: PartialOrder, B: PartialOrder](
    lower: A -> B,
    upper: B -> A
) -> Bool {
    is_galois_connection(lower, upper) and forall(a: A) {
        upper(lower(a)) = a
    }
}

/// A Galois insertion is a Galois connection.
theorem galois_insertion_is_connection[A: PartialOrder, B: PartialOrder](
    lower: A -> B,
    upper: B -> A
) {
    is_galois_insertion(lower, upper) implies is_galois_connection(lower, upper)
} by {
    if is_galois_insertion(lower, upper) {
        is_galois_connection(lower, upper)
    }
}

/// A Galois coinsertion is a Galois connection.
theorem galois_coinsertion_is_connection[A: PartialOrder, B: PartialOrder](
    lower: A -> B,
    upper: B -> A
) {
    is_galois_coinsertion(lower, upper) implies is_galois_connection(lower, upper)
} by {
    if is_galois_coinsertion(lower, upper) {
        is_galois_connection(lower, upper)
    }
}

/// The counit of a Galois insertion is equality.
theorem galois_insertion_counit_eq[A: PartialOrder, B: PartialOrder](
    lower: A -> B,
    upper: B -> A,
    b: B
) {
    is_galois_insertion(lower, upper) implies lower(upper(b)) = b
} by {
    if is_galois_insertion(lower, upper) {
        is_galois_insertion(lower, upper) =
            (is_galois_connection(lower, upper) and forall(y: B) {
                lower(upper(y)) = y
            })
        lower(upper(b)) = b
    }
}

/// The unit of a Galois coinsertion is equality.
theorem galois_coinsertion_unit_eq[A: PartialOrder, B: PartialOrder](
    lower: A -> B,
    upper: B -> A,
    a: A
) {
    is_galois_coinsertion(lower, upper) implies upper(lower(a)) = a
} by {
    if is_galois_coinsertion(lower, upper) {
        is_galois_coinsertion(lower, upper) =
            (is_galois_connection(lower, upper) and forall(x: A) {
                upper(lower(x)) = x
            })
        upper(lower(a)) = a
    }
}

/// The kernel of a Galois insertion is the identity.
theorem galois_insertion_kernel_eq_self[A: PartialOrder, B: PartialOrder](
    lower: A -> B,
    upper: B -> A,
    b: B
) {
    is_galois_insertion(lower, upper) implies galois_kernel(lower, upper, b) = b
} by {
    if is_galois_insertion(lower, upper) {
        galois_insertion_counit_eq(lower, upper, b)
        galois_kernel_at(lower, upper, b)
        galois_kernel(lower, upper, b) = b
    }
}

/// The closure of a Galois coinsertion is the identity.
theorem galois_coinsertion_closure_eq_self[A: PartialOrder, B: PartialOrder](
    lower: A -> B,
    upper: B -> A,
    a: A
) {
    is_galois_coinsertion(lower, upper) implies galois_closure(lower, upper, a) = a
} by {
    if is_galois_coinsertion(lower, upper) {
        galois_coinsertion_unit_eq(lower, upper, a)
        galois_closure_at(lower, upper, a)
        galois_closure(lower, upper, a) = a
    }
}

/// The meet transported to the upper side of a Galois insertion.
define galois_insertion_meet[A: MeetSemilattice, B: PartialOrder](
    lower: A -> B,
    upper: B -> A,
    x: B,
    y: B
) -> B {
    lower(upper(x).meet(upper(y)))
}

/// The join transported to the upper side of a Galois insertion.
define galois_insertion_join[A: JoinSemilattice, B: PartialOrder](
    lower: A -> B,
    upper: B -> A,
    x: B,
    y: B
) -> B {
    lower(upper(x).join(upper(y)))
}

/// The transported meet is below its left argument.
theorem galois_insertion_meet_lte_left[A: MeetSemilattice, B: PartialOrder](
    lower: A -> B,
    upper: B -> A,
    x: B,
    y: B
) {
    is_galois_insertion(lower, upper) implies
    galois_insertion_meet(lower, upper, x, y) <= x
} by {
    if is_galois_insertion(lower, upper) {
        is_galois_connection(lower, upper)
        meet_lte_left(upper(x), upper(y))
        galois_connection_lower_le(lower, upper, upper(x).meet(upper(y)), x)
        galois_insertion_meet(lower, upper, x, y) <= x
    }
}

/// The transported meet is below its right argument.
theorem galois_insertion_meet_lte_right[A: MeetSemilattice, B: PartialOrder](
    lower: A -> B,
    upper: B -> A,
    x: B,
    y: B
) {
    is_galois_insertion(lower, upper) implies
    galois_insertion_meet(lower, upper, x, y) <= y
} by {
    if is_galois_insertion(lower, upper) {
        is_galois_connection(lower, upper)
        meet_lte_right(upper(x), upper(y))
        galois_connection_lower_le(lower, upper, upper(x).meet(upper(y)), y)
        galois_insertion_meet(lower, upper, x, y) <= y
    }
}

/// Every common lower bound is below the transported meet.
theorem lte_galois_insertion_meet_of_bounds[A: MeetSemilattice, B: PartialOrder](
    lower: A -> B,
    upper: B -> A,
    z: B,
    x: B,
    y: B
) {
    is_galois_insertion(lower, upper) and z <= x and z <= y implies
    z <= galois_insertion_meet(lower, upper, x, y)
} by {
    if is_galois_insertion(lower, upper) and z <= x and z <= y {
        galois_insertion_is_connection(lower, upper)
        galois_connection_upper_monotone(lower, upper)
        monotone_step(upper, z, x)
        monotone_step(upper, z, y)
        upper(z) <= upper(y)
        lte_meet_of_bounds(upper(z), upper(x), upper(y))
        galois_connection_lower_monotone(lower, upper)
        monotone_step(lower, upper(z), upper(x).meet(upper(y)))
        lower(upper(z)) <= lower(upper(x).meet(upper(y)))
        galois_insertion_counit_eq(lower, upper, z)
        z <= galois_insertion_meet(lower, upper, x, y)
    }
}

/// The transported meet is characterized by its lower bounds.
theorem lte_galois_insertion_meet_iff[A: MeetSemilattice, B: PartialOrder](
    lower: A -> B,
    upper: B -> A,
    z: B,
    x: B,
    y: B
) {
    is_galois_insertion(lower, upper) implies
    (z <= galois_insertion_meet(lower, upper, x, y) = (z <= x and z <= y))
} by {
    if is_galois_insertion(lower, upper) {
        if z <= galois_insertion_meet(lower, upper, x, y) {
            galois_insertion_meet_lte_left(lower, upper, x, y)
            lte_trans(z, galois_insertion_meet(lower, upper, x, y), x)
            galois_insertion_meet_lte_right(lower, upper, x, y)
            lte_trans(z, galois_insertion_meet(lower, upper, x, y), y)
            z <= y
            z <= x and z <= y
        }
        if z <= x and z <= y {
            lte_galois_insertion_meet_of_bounds(lower, upper, z, x, y)
            z <= galois_insertion_meet(lower, upper, x, y)
        }
        z <= galois_insertion_meet(lower, upper, x, y) = (z <= x and z <= y)
    }
}

/// The left argument is below the transported join.
theorem galois_insertion_lte_join_left[A: JoinSemilattice, B: PartialOrder](
    lower: A -> B,
    upper: B -> A,
    x: B,
    y: B
) {
    is_galois_insertion(lower, upper) implies
    x <= galois_insertion_join(lower, upper, x, y)
} by {
    if is_galois_insertion(lower, upper) {
        lte_join_left(upper(x), upper(y))
        galois_connection_lower_monotone(lower, upper)
        is_monotone(lower)
        monotone_step(lower, upper(x), upper(x).join(upper(y)))
        galois_insertion_counit_eq(lower, upper, x)
        lower(upper(x)) = x
        x <= galois_insertion_join(lower, upper, x, y)
    }
}

/// The right argument is below the transported join.
theorem galois_insertion_lte_join_right[A: JoinSemilattice, B: PartialOrder](
    lower: A -> B,
    upper: B -> A,
    x: B,
    y: B
) {
    is_galois_insertion(lower, upper) implies
    y <= galois_insertion_join(lower, upper, x, y)
} by {
    if is_galois_insertion(lower, upper) {
        lte_join_right(upper(x), upper(y))
        galois_connection_lower_monotone(lower, upper)
        is_monotone(lower)
        monotone_step(lower, upper(y), upper(x).join(upper(y)))
        galois_insertion_counit_eq(lower, upper, y)
        lower(upper(y)) = y
        y <= galois_insertion_join(lower, upper, x, y)
    }
}

/// The transported join is below every common upper bound.
theorem galois_insertion_join_lte_of_bounds[A: JoinSemilattice, B: PartialOrder](
    lower: A -> B,
    upper: B -> A,
    x: B,
    y: B,
    z: B
) {
    is_galois_insertion(lower, upper) and x <= z and y <= z implies
    galois_insertion_join(lower, upper, x, y) <= z
} by {
    if is_galois_insertion(lower, upper) and x <= z and y <= z {
        galois_insertion_is_connection(lower, upper)
        galois_connection_upper_monotone(lower, upper)
        monotone_step(upper, x, z)
        monotone_step(upper, y, z)
        upper(y) <= upper(z)
        join_lte_of_bounds(upper(x), upper(y), upper(z))
        galois_connection_lower_le(lower, upper, upper(x).join(upper(y)), z)
        lower(upper(x).join(upper(y))) <= z
        galois_insertion_join(lower, upper, x, y) <= z
    }
}

/// The transported join is characterized by its upper bounds.
theorem galois_insertion_join_lte_iff[A: JoinSemilattice, B: PartialOrder](
    lower: A -> B,
    upper: B -> A,
    x: B,
    y: B,
    z: B
) {
    is_galois_insertion(lower, upper) implies
    (galois_insertion_join(lower, upper, x, y) <= z = (x <= z and y <= z))
} by {
    if is_galois_insertion(lower, upper) {
        if galois_insertion_join(lower, upper, x, y) <= z {
            galois_insertion_lte_join_left(lower, upper, x, y)
            lte_trans(x, galois_insertion_join(lower, upper, x, y), z)
            galois_insertion_lte_join_right(lower, upper, x, y)
            lte_trans(y, galois_insertion_join(lower, upper, x, y), z)
            y <= z
            x <= z and y <= z
        }
        if x <= z and y <= z {
            galois_insertion_join_lte_of_bounds(lower, upper, x, y, z)
            galois_insertion_join(lower, upper, x, y) <= z
        }
        galois_insertion_join(lower, upper, x, y) <= z = (x <= z and y <= z)
    }
}

/// The transported meet is commutative.
theorem galois_insertion_meet_comm[A: MeetSemilattice, B: PartialOrder](
    lower: A -> B,
    upper: B -> A,
    x: B,
    y: B
) {
    is_galois_insertion(lower, upper) implies
    galois_insertion_meet(lower, upper, x, y) = galois_insertion_meet(lower, upper, y, x)
} by {
    if is_galois_insertion(lower, upper) {
        galois_insertion_meet_lte_left(lower, upper, x, y)
        galois_insertion_meet_lte_right(lower, upper, x, y)
        galois_insertion_meet(lower, upper, x, y) <= y
        lte_galois_insertion_meet_of_bounds(
            lower, upper, galois_insertion_meet(lower, upper, x, y), y, x
        )

        galois_insertion_meet_lte_left(lower, upper, y, x)
        galois_insertion_meet_lte_right(lower, upper, y, x)
        galois_insertion_meet(lower, upper, y, x) <= x
        lte_galois_insertion_meet_of_bounds(
            lower, upper, galois_insertion_meet(lower, upper, y, x), x, y
        )
        galois_insertion_meet(lower, upper, y, x) <= galois_insertion_meet(lower, upper, x, y)
        lte_antisymm(galois_insertion_meet(lower, upper, x, y), galois_insertion_meet(lower, upper, y, x))
        galois_insertion_meet(lower, upper, x, y) = galois_insertion_meet(lower, upper, y, x)
    }
}

/// The transported join is commutative.
theorem galois_insertion_join_comm[A: JoinSemilattice, B: PartialOrder](
    lower: A -> B,
    upper: B -> A,
    x: B,
    y: B
) {
    is_galois_insertion(lower, upper) implies
    galois_insertion_join(lower, upper, x, y) = galois_insertion_join(lower, upper, y, x)
} by {
    if is_galois_insertion(lower, upper) {
        galois_insertion_lte_join_left(lower, upper, x, y)
        galois_insertion_lte_join_right(lower, upper, x, y)
        y <= galois_insertion_join(lower, upper, x, y)
        galois_insertion_join_lte_of_bounds(
            lower, upper, y, x, galois_insertion_join(lower, upper, x, y)
        )

        galois_insertion_lte_join_left(lower, upper, y, x)
        galois_insertion_lte_join_right(lower, upper, y, x)
        x <= galois_insertion_join(lower, upper, y, x)
        galois_insertion_join_lte_of_bounds(
            lower, upper, x, y, galois_insertion_join(lower, upper, y, x)
        )
        galois_insertion_join(lower, upper, x, y) <= galois_insertion_join(lower, upper, y, x)
        lte_antisymm(galois_insertion_join(lower, upper, x, y), galois_insertion_join(lower, upper, y, x))
        galois_insertion_join(lower, upper, x, y) = galois_insertion_join(lower, upper, y, x)
    }
}

/// The transported meet is idempotent.
theorem galois_insertion_meet_idem[A: MeetSemilattice, B: PartialOrder](
    lower: A -> B,
    upper: B -> A,
    x: B
) {
    is_galois_insertion(lower, upper) implies
    galois_insertion_meet(lower, upper, x, x) = x
} by {
    if is_galois_insertion(lower, upper) {
        galois_insertion_meet_lte_left(lower, upper, x, x)
        lte_refl(x)
        lte_galois_insertion_meet_of_bounds(lower, upper, x, x, x)
        x <= galois_insertion_meet(lower, upper, x, x)
        lte_antisymm(galois_insertion_meet(lower, upper, x, x), x)
        galois_insertion_meet(lower, upper, x, x) = x
    }
}

/// The transported join is idempotent.
theorem galois_insertion_join_idem[A: JoinSemilattice, B: PartialOrder](
    lower: A -> B,
    upper: B -> A,
    x: B
) {
    is_galois_insertion(lower, upper) implies
    galois_insertion_join(lower, upper, x, x) = x
} by {
    if is_galois_insertion(lower, upper) {
        galois_insertion_lte_join_left(lower, upper, x, x)
        lte_refl(x)
        galois_insertion_join_lte_of_bounds(lower, upper, x, x, x)
        galois_insertion_join(lower, upper, x, x) <= x
        lte_antisymm(galois_insertion_join(lower, upper, x, x), x)
        galois_insertion_join(lower, upper, x, x) = x
    }
}

/// The transported meet is associative.
theorem galois_insertion_meet_assoc[A: MeetSemilattice, B: PartialOrder](
    lower: A -> B,
    upper: B -> A,
    x: B,
    y: B,
    z: B
) {
    is_galois_insertion(lower, upper) implies
    galois_insertion_meet(lower, upper, galois_insertion_meet(lower, upper, x, y), z) =
    galois_insertion_meet(lower, upper, x, galois_insertion_meet(lower, upper, y, z))
} by {
    if is_galois_insertion(lower, upper) {
        let xy = galois_insertion_meet(lower, upper, x, y)
        let yz = galois_insertion_meet(lower, upper, y, z)
        let lhs = galois_insertion_meet(lower, upper, xy, z)
        let rhs = galois_insertion_meet(lower, upper, x, yz)

        galois_insertion_meet_lte_left(lower, upper, xy, z)
        galois_insertion_meet_lte_left(lower, upper, x, y)
        xy <= x
        lte_trans(lhs, xy, x)
        galois_insertion_meet_lte_right(lower, upper, xy, z)
        galois_insertion_meet_lte_left(lower, upper, xy, z)
        lhs <= xy
        galois_insertion_meet_lte_right(lower, upper, x, y)
        lte_trans(lhs, xy, y)
        lhs <= y
        lte_galois_insertion_meet_of_bounds(lower, upper, lhs, y, z)
        lhs <= yz
        lte_galois_insertion_meet_of_bounds(lower, upper, lhs, x, yz)
        lhs <= rhs

        galois_insertion_meet_lte_left(lower, upper, x, yz)
        galois_insertion_meet_lte_right(lower, upper, x, yz)
        galois_insertion_meet_lte_left(lower, upper, y, z)
        yz <= y
        lte_trans(rhs, yz, y)
        rhs <= y
        galois_insertion_meet_lte_right(lower, upper, y, z)
        yz <= z
        lte_trans(rhs, yz, z)
        rhs <= z
        lte_galois_insertion_meet_of_bounds(lower, upper, rhs, x, y)
        rhs <= xy
        lte_galois_insertion_meet_of_bounds(lower, upper, rhs, xy, z)

        lte_antisymm(lhs, rhs)
        lhs = rhs
        galois_insertion_meet(lower, upper, galois_insertion_meet(lower, upper, x, y), z) =
        galois_insertion_meet(lower, upper, x, galois_insertion_meet(lower, upper, y, z))
    }
}

/// The transported join is associative.
theorem galois_insertion_join_assoc[A: JoinSemilattice, B: PartialOrder](
    lower: A -> B,
    upper: B -> A,
    x: B,
    y: B,
    z: B
) {
    is_galois_insertion(lower, upper) implies
    galois_insertion_join(lower, upper, galois_insertion_join(lower, upper, x, y), z) =
    galois_insertion_join(lower, upper, x, galois_insertion_join(lower, upper, y, z))
} by {
    if is_galois_insertion(lower, upper) {
        let xy = galois_insertion_join(lower, upper, x, y)
        let yz = galois_insertion_join(lower, upper, y, z)
        let lhs = galois_insertion_join(lower, upper, xy, z)
        let rhs = galois_insertion_join(lower, upper, x, yz)

        galois_insertion_lte_join_left(lower, upper, xy, z)
        galois_insertion_lte_join_left(lower, upper, x, y)
        x <= xy
        lte_trans(x, xy, lhs)
        galois_insertion_lte_join_right(lower, upper, xy, z)
        galois_insertion_lte_join_left(lower, upper, xy, z)
        xy <= lhs
        galois_insertion_lte_join_right(lower, upper, x, y)
        lte_trans(y, xy, lhs)
        y <= lhs
        galois_insertion_join_lte_of_bounds(lower, upper, y, z, lhs)
        yz <= lhs
        galois_insertion_join_lte_of_bounds(lower, upper, x, yz, lhs)
        rhs <= lhs

        galois_insertion_lte_join_left(lower, upper, x, yz)
        galois_insertion_lte_join_right(lower, upper, x, yz)
        galois_insertion_lte_join_left(lower, upper, y, z)
        y <= yz
        lte_trans(y, yz, rhs)
        y <= rhs
        galois_insertion_lte_join_right(lower, upper, y, z)
        z <= yz
        lte_trans(z, yz, rhs)
        z <= rhs
        galois_insertion_join_lte_of_bounds(lower, upper, x, y, rhs)
        xy <= rhs
        galois_insertion_join_lte_of_bounds(lower, upper, xy, z, rhs)

        lte_antisymm(lhs, rhs)
        lhs = rhs
        galois_insertion_join(lower, upper, galois_insertion_join(lower, upper, x, y), z) =
        galois_insertion_join(lower, upper, x, galois_insertion_join(lower, upper, y, z))
    }
}

/// The meet transported to the lower side of a Galois coinsertion.
define galois_coinsertion_meet[A: PartialOrder, B: MeetSemilattice](
    lower: A -> B,
    upper: B -> A,
    x: A,
    y: A
) -> A {
    upper(lower(x).meet(lower(y)))
}

/// The join transported to the lower side of a Galois coinsertion.
define galois_coinsertion_join[A: PartialOrder, B: JoinSemilattice](
    lower: A -> B,
    upper: B -> A,
    x: A,
    y: A
) -> A {
    upper(lower(x).join(lower(y)))
}

/// The transported meet is below its left argument.
theorem galois_coinsertion_meet_lte_left[A: PartialOrder, B: MeetSemilattice](
    lower: A -> B,
    upper: B -> A,
    x: A,
    y: A
) {
    is_galois_coinsertion(lower, upper) implies
    galois_coinsertion_meet(lower, upper, x, y) <= x
} by {
    if is_galois_coinsertion(lower, upper) {
        galois_coinsertion_is_connection(lower, upper)
        galois_connection_lower_monotone(lower, upper)
        galois_connection_upper_monotone(lower, upper)
        forall(a: A, b: A) {
            if lower(a) <= lower(b) {
                monotone_step(upper, lower(a), lower(b))
                upper(lower(a)) <= upper(lower(b))
                galois_coinsertion_unit_eq(lower, upper, a)
                galois_coinsertion_unit_eq(lower, upper, b)
                upper(lower(a)) = a
                upper(lower(b)) = b
                a <= b
            }
            if a <= b {
                monotone_step(lower, a, b)
                lower(a) <= lower(b)
            }
            lower(a) <= lower(b) = (a <= b)
        }
        let m = lower(x).meet(lower(y))
        galois_connection_counit(lower, upper, m)
        meet_lte_left(lower(x), lower(y))
        m <= lower(x)
        lte_trans(lower(upper(m)), m, lower(x))
        lower(upper(m)) <= lower(x)
        order_embedding_reflects_lte(lower, upper(m), x)
        galois_coinsertion_meet(lower, upper, x, y) <= x
    }
}

/// The transported meet is below its right argument.
theorem galois_coinsertion_meet_lte_right[A: PartialOrder, B: MeetSemilattice](
    lower: A -> B,
    upper: B -> A,
    x: A,
    y: A
) {
    is_galois_coinsertion(lower, upper) implies
    galois_coinsertion_meet(lower, upper, x, y) <= y
} by {
    if is_galois_coinsertion(lower, upper) {
        galois_coinsertion_is_connection(lower, upper)
        galois_connection_lower_monotone(lower, upper)
        galois_connection_upper_monotone(lower, upper)
        forall(a: A, b: A) {
            if lower(a) <= lower(b) {
                monotone_step(upper, lower(a), lower(b))
                upper(lower(a)) <= upper(lower(b))
                galois_coinsertion_unit_eq(lower, upper, a)
                galois_coinsertion_unit_eq(lower, upper, b)
                upper(lower(a)) = a
                upper(lower(b)) = b
                a <= b
            }
            if a <= b {
                monotone_step(lower, a, b)
                lower(a) <= lower(b)
            }
            lower(a) <= lower(b) = (a <= b)
        }
        let m = lower(x).meet(lower(y))
        galois_connection_counit(lower, upper, m)
        meet_lte_right(lower(x), lower(y))
        m <= lower(y)
        lte_trans(lower(upper(m)), m, lower(y))
        lower(upper(m)) <= lower(y)
        order_embedding_reflects_lte(lower, upper(m), y)
        galois_coinsertion_meet(lower, upper, x, y) <= y
    }
}

/// Every common lower bound is below the transported meet.
theorem lte_galois_coinsertion_meet_of_bounds[A: PartialOrder, B: MeetSemilattice](
    lower: A -> B,
    upper: B -> A,
    z: A,
    x: A,
    y: A
) {
    is_galois_coinsertion(lower, upper) and z <= x and z <= y implies
    z <= galois_coinsertion_meet(lower, upper, x, y)
} by {
    if is_galois_coinsertion(lower, upper) and z <= x and z <= y {
        galois_coinsertion_is_connection(lower, upper)
        galois_connection_lower_monotone(lower, upper)
        monotone_step(lower, z, x)
        monotone_step(lower, z, y)
        lower(z) <= lower(y)
        lte_meet_of_bounds(lower(z), lower(x), lower(y))
        galois_connection_le_upper(lower, upper, z, lower(x).meet(lower(y)))
        z <= upper(lower(x).meet(lower(y)))
        z <= galois_coinsertion_meet(lower, upper, x, y)
    }
}

/// The transported meet is characterized by its lower bounds.
theorem lte_galois_coinsertion_meet_iff[A: PartialOrder, B: MeetSemilattice](
    lower: A -> B,
    upper: B -> A,
    z: A,
    x: A,
    y: A
) {
    is_galois_coinsertion(lower, upper) implies
    (z <= galois_coinsertion_meet(lower, upper, x, y) = (z <= x and z <= y))
} by {
    if is_galois_coinsertion(lower, upper) {
        if z <= galois_coinsertion_meet(lower, upper, x, y) {
            galois_coinsertion_meet_lte_left(lower, upper, x, y)
            lte_trans(z, galois_coinsertion_meet(lower, upper, x, y), x)
            galois_coinsertion_meet_lte_right(lower, upper, x, y)
            lte_trans(z, galois_coinsertion_meet(lower, upper, x, y), y)
            z <= y
            z <= x and z <= y
        }
        if z <= x and z <= y {
            lte_galois_coinsertion_meet_of_bounds(lower, upper, z, x, y)
            z <= galois_coinsertion_meet(lower, upper, x, y)
        }
        z <= galois_coinsertion_meet(lower, upper, x, y) = (z <= x and z <= y)
    }
}

/// The left argument is below the transported join.
theorem galois_coinsertion_lte_join_left[A: PartialOrder, B: JoinSemilattice](
    lower: A -> B,
    upper: B -> A,
    x: A,
    y: A
) {
    is_galois_coinsertion(lower, upper) implies
    x <= galois_coinsertion_join(lower, upper, x, y)
} by {
    if is_galois_coinsertion(lower, upper) {
        galois_coinsertion_is_connection(lower, upper)
        lte_join_left(lower(x), lower(y))
        galois_connection_le_upper(lower, upper, x, lower(x).join(lower(y)))
        x <= galois_coinsertion_join(lower, upper, x, y)
    }
}

/// The right argument is below the transported join.
theorem galois_coinsertion_lte_join_right[A: PartialOrder, B: JoinSemilattice](
    lower: A -> B,
    upper: B -> A,
    x: A,
    y: A
) {
    is_galois_coinsertion(lower, upper) implies
    y <= galois_coinsertion_join(lower, upper, x, y)
} by {
    if is_galois_coinsertion(lower, upper) {
        galois_coinsertion_is_connection(lower, upper)
        lte_join_right(lower(x), lower(y))
        galois_connection_le_upper(lower, upper, y, lower(x).join(lower(y)))
        y <= galois_coinsertion_join(lower, upper, x, y)
    }
}

/// The transported join is below every common upper bound.
theorem galois_coinsertion_join_lte_of_bounds[A: PartialOrder, B: JoinSemilattice](
    lower: A -> B,
    upper: B -> A,
    x: A,
    y: A,
    z: A
) {
    is_galois_coinsertion(lower, upper) and x <= z and y <= z implies
    galois_coinsertion_join(lower, upper, x, y) <= z
} by {
    if is_galois_coinsertion(lower, upper) and x <= z and y <= z {
        galois_coinsertion_is_connection(lower, upper)
        galois_connection_lower_monotone(lower, upper)
        galois_connection_upper_monotone(lower, upper)
        forall(a: A, b: A) {
            if lower(a) <= lower(b) {
                monotone_step(upper, lower(a), lower(b))
                upper(lower(a)) <= upper(lower(b))
                galois_coinsertion_unit_eq(lower, upper, a)
                galois_coinsertion_unit_eq(lower, upper, b)
                upper(lower(a)) = a
                upper(lower(b)) = b
                a <= b
            }
            if a <= b {
                monotone_step(lower, a, b)
                lower(a) <= lower(b)
            }
            lower(a) <= lower(b) = (a <= b)
        }
        monotone_step(lower, x, z)
        monotone_step(lower, y, z)
        join_lte_of_bounds(lower(x), lower(y), lower(z))
        let j = lower(x).join(lower(y))
        galois_connection_counit(lower, upper, j)
        lower(upper(j)) <= j
        lte_trans(lower(upper(j)), j, lower(z))
        lower(upper(j)) <= lower(z)
        order_embedding_reflects_lte(lower, upper(j), z)
        galois_coinsertion_join(lower, upper, x, y) <= z
    }
}

/// The transported join is characterized by its upper bounds.
theorem galois_coinsertion_join_lte_iff[A: PartialOrder, B: JoinSemilattice](
    lower: A -> B,
    upper: B -> A,
    x: A,
    y: A,
    z: A
) {
    is_galois_coinsertion(lower, upper) implies
    (galois_coinsertion_join(lower, upper, x, y) <= z = (x <= z and y <= z))
} by {
    if is_galois_coinsertion(lower, upper) {
        if galois_coinsertion_join(lower, upper, x, y) <= z {
            galois_coinsertion_lte_join_left(lower, upper, x, y)
            lte_trans(x, galois_coinsertion_join(lower, upper, x, y), z)
            galois_coinsertion_lte_join_right(lower, upper, x, y)
            lte_trans(y, galois_coinsertion_join(lower, upper, x, y), z)
            y <= z
            x <= z and y <= z
        }
        if x <= z and y <= z {
            galois_coinsertion_join_lte_of_bounds(lower, upper, x, y, z)
            galois_coinsertion_join(lower, upper, x, y) <= z
        }
        galois_coinsertion_join(lower, upper, x, y) <= z = (x <= z and y <= z)
    }
}

/// The transported meet is commutative.
theorem galois_coinsertion_meet_comm[A: PartialOrder, B: MeetSemilattice](
    lower: A -> B,
    upper: B -> A,
    x: A,
    y: A
) {
    is_galois_coinsertion(lower, upper) implies
    galois_coinsertion_meet(lower, upper, x, y) = galois_coinsertion_meet(lower, upper, y, x)
} by {
    if is_galois_coinsertion(lower, upper) {
        galois_coinsertion_meet_lte_left(lower, upper, x, y)
        galois_coinsertion_meet_lte_right(lower, upper, x, y)
        galois_coinsertion_meet(lower, upper, x, y) <= y
        lte_galois_coinsertion_meet_of_bounds(
            lower, upper, galois_coinsertion_meet(lower, upper, x, y), y, x
        )

        galois_coinsertion_meet_lte_left(lower, upper, y, x)
        galois_coinsertion_meet_lte_right(lower, upper, y, x)
        galois_coinsertion_meet(lower, upper, y, x) <= x
        lte_galois_coinsertion_meet_of_bounds(
            lower, upper, galois_coinsertion_meet(lower, upper, y, x), x, y
        )
        galois_coinsertion_meet(lower, upper, y, x) <= galois_coinsertion_meet(lower, upper, x, y)
        lte_antisymm(galois_coinsertion_meet(lower, upper, x, y), galois_coinsertion_meet(lower, upper, y, x))
        galois_coinsertion_meet(lower, upper, x, y) = galois_coinsertion_meet(lower, upper, y, x)
    }
}

/// The transported join is commutative.
theorem galois_coinsertion_join_comm[A: PartialOrder, B: JoinSemilattice](
    lower: A -> B,
    upper: B -> A,
    x: A,
    y: A
) {
    is_galois_coinsertion(lower, upper) implies
    galois_coinsertion_join(lower, upper, x, y) = galois_coinsertion_join(lower, upper, y, x)
} by {
    if is_galois_coinsertion(lower, upper) {
        galois_coinsertion_lte_join_left(lower, upper, x, y)
        galois_coinsertion_lte_join_right(lower, upper, x, y)
        y <= galois_coinsertion_join(lower, upper, x, y)
        galois_coinsertion_join_lte_of_bounds(
            lower, upper, y, x, galois_coinsertion_join(lower, upper, x, y)
        )

        galois_coinsertion_lte_join_left(lower, upper, y, x)
        galois_coinsertion_lte_join_right(lower, upper, y, x)
        x <= galois_coinsertion_join(lower, upper, y, x)
        galois_coinsertion_join_lte_of_bounds(
            lower, upper, x, y, galois_coinsertion_join(lower, upper, y, x)
        )
        galois_coinsertion_join(lower, upper, x, y) <= galois_coinsertion_join(lower, upper, y, x)
        lte_antisymm(galois_coinsertion_join(lower, upper, x, y), galois_coinsertion_join(lower, upper, y, x))
        galois_coinsertion_join(lower, upper, x, y) = galois_coinsertion_join(lower, upper, y, x)
    }
}

/// The transported meet is idempotent.
theorem galois_coinsertion_meet_idem[A: PartialOrder, B: MeetSemilattice](
    lower: A -> B,
    upper: B -> A,
    x: A
) {
    is_galois_coinsertion(lower, upper) implies
    galois_coinsertion_meet(lower, upper, x, x) = x
} by {
    if is_galois_coinsertion(lower, upper) {
        galois_coinsertion_meet_lte_left(lower, upper, x, x)
        lte_refl(x)
        lte_galois_coinsertion_meet_of_bounds(lower, upper, x, x, x)
        x <= galois_coinsertion_meet(lower, upper, x, x)
        lte_antisymm(galois_coinsertion_meet(lower, upper, x, x), x)
        galois_coinsertion_meet(lower, upper, x, x) = x
    }
}

/// The transported join is idempotent.
theorem galois_coinsertion_join_idem[A: PartialOrder, B: JoinSemilattice](
    lower: A -> B,
    upper: B -> A,
    x: A
) {
    is_galois_coinsertion(lower, upper) implies
    galois_coinsertion_join(lower, upper, x, x) = x
} by {
    if is_galois_coinsertion(lower, upper) {
        galois_coinsertion_lte_join_left(lower, upper, x, x)
        lte_refl(x)
        galois_coinsertion_join_lte_of_bounds(lower, upper, x, x, x)
        galois_coinsertion_join(lower, upper, x, x) <= x
        lte_antisymm(galois_coinsertion_join(lower, upper, x, x), x)
        galois_coinsertion_join(lower, upper, x, x) = x
    }
}

/// The transported meet is associative.
theorem galois_coinsertion_meet_assoc[A: PartialOrder, B: MeetSemilattice](
    lower: A -> B,
    upper: B -> A,
    x: A,
    y: A,
    z: A
) {
    is_galois_coinsertion(lower, upper) implies
    galois_coinsertion_meet(lower, upper, galois_coinsertion_meet(lower, upper, x, y), z) =
    galois_coinsertion_meet(lower, upper, x, galois_coinsertion_meet(lower, upper, y, z))
} by {
    if is_galois_coinsertion(lower, upper) {
        let xy = galois_coinsertion_meet(lower, upper, x, y)
        let yz = galois_coinsertion_meet(lower, upper, y, z)
        let lhs = galois_coinsertion_meet(lower, upper, xy, z)
        let rhs = galois_coinsertion_meet(lower, upper, x, yz)

        galois_coinsertion_meet_lte_left(lower, upper, xy, z)
        galois_coinsertion_meet_lte_left(lower, upper, x, y)
        xy <= x
        lte_trans(lhs, xy, x)
        galois_coinsertion_meet_lte_right(lower, upper, xy, z)
        galois_coinsertion_meet_lte_left(lower, upper, xy, z)
        lhs <= xy
        galois_coinsertion_meet_lte_right(lower, upper, x, y)
        lte_trans(lhs, xy, y)
        lhs <= y
        lte_galois_coinsertion_meet_of_bounds(lower, upper, lhs, y, z)
        lhs <= yz
        lte_galois_coinsertion_meet_of_bounds(lower, upper, lhs, x, yz)
        lhs <= rhs

        galois_coinsertion_meet_lte_left(lower, upper, x, yz)
        galois_coinsertion_meet_lte_right(lower, upper, x, yz)
        galois_coinsertion_meet_lte_left(lower, upper, y, z)
        yz <= y
        lte_trans(rhs, yz, y)
        rhs <= y
        galois_coinsertion_meet_lte_right(lower, upper, y, z)
        yz <= z
        lte_trans(rhs, yz, z)
        rhs <= z
        lte_galois_coinsertion_meet_of_bounds(lower, upper, rhs, x, y)
        rhs <= xy
        lte_galois_coinsertion_meet_of_bounds(lower, upper, rhs, xy, z)

        lte_antisymm(lhs, rhs)
        lhs = rhs
        galois_coinsertion_meet(lower, upper, galois_coinsertion_meet(lower, upper, x, y), z) =
        galois_coinsertion_meet(lower, upper, x, galois_coinsertion_meet(lower, upper, y, z))
    }
}

/// The transported join is associative.
theorem galois_coinsertion_join_assoc[A: PartialOrder, B: JoinSemilattice](
    lower: A -> B,
    upper: B -> A,
    x: A,
    y: A,
    z: A
) {
    is_galois_coinsertion(lower, upper) implies
    galois_coinsertion_join(lower, upper, galois_coinsertion_join(lower, upper, x, y), z) =
    galois_coinsertion_join(lower, upper, x, galois_coinsertion_join(lower, upper, y, z))
} by {
    if is_galois_coinsertion(lower, upper) {
        let xy = galois_coinsertion_join(lower, upper, x, y)
        let yz = galois_coinsertion_join(lower, upper, y, z)
        let lhs = galois_coinsertion_join(lower, upper, xy, z)
        let rhs = galois_coinsertion_join(lower, upper, x, yz)

        galois_coinsertion_lte_join_left(lower, upper, xy, z)
        galois_coinsertion_lte_join_left(lower, upper, x, y)
        x <= xy
        lte_trans(x, xy, lhs)
        galois_coinsertion_lte_join_right(lower, upper, xy, z)
        galois_coinsertion_lte_join_left(lower, upper, xy, z)
        xy <= lhs
        galois_coinsertion_lte_join_right(lower, upper, x, y)
        lte_trans(y, xy, lhs)
        y <= lhs
        galois_coinsertion_join_lte_of_bounds(lower, upper, y, z, lhs)
        yz <= lhs
        galois_coinsertion_join_lte_of_bounds(lower, upper, x, yz, lhs)
        rhs <= lhs

        galois_coinsertion_lte_join_left(lower, upper, x, yz)
        galois_coinsertion_lte_join_right(lower, upper, x, yz)
        galois_coinsertion_lte_join_left(lower, upper, y, z)
        y <= yz
        lte_trans(y, yz, rhs)
        y <= rhs
        galois_coinsertion_lte_join_right(lower, upper, y, z)
        z <= yz
        lte_trans(z, yz, rhs)
        z <= rhs
        galois_coinsertion_join_lte_of_bounds(lower, upper, x, y, rhs)
        xy <= rhs
        galois_coinsertion_join_lte_of_bounds(lower, upper, xy, z, rhs)

        lte_antisymm(lhs, rhs)
        lhs = rhs
        galois_coinsertion_join(lower, upper, galois_coinsertion_join(lower, upper, x, y), z) =
        galois_coinsertion_join(lower, upper, x, galois_coinsertion_join(lower, upper, y, z))
    }
}

/// The transported meet is monotone in both arguments.
theorem galois_coinsertion_meet_lte_meet[A: PartialOrder, B: MeetSemilattice](
    lower: A -> B,
    upper: B -> A,
    a: A,
    b: A,
    c: A,
    d: A
) {
    is_galois_coinsertion(lower, upper) and a <= b and c <= d implies
    galois_coinsertion_meet(lower, upper, a, c) <= galois_coinsertion_meet(lower, upper, b, d)
} by {
    if is_galois_coinsertion(lower, upper) and a <= b and c <= d {
        galois_coinsertion_meet_lte_left(lower, upper, a, c)
        lte_trans(galois_coinsertion_meet(lower, upper, a, c), a, b)
        galois_coinsertion_meet_lte_right(lower, upper, a, c)
        lte_trans(galois_coinsertion_meet(lower, upper, a, c), c, d)
        galois_coinsertion_meet(lower, upper, a, c) <= d
        lte_galois_coinsertion_meet_of_bounds(
            lower, upper, galois_coinsertion_meet(lower, upper, a, c), b, d
        )
        galois_coinsertion_meet(lower, upper, a, c) <= galois_coinsertion_meet(lower, upper, b, d)
    }
}

/// The transported join is monotone in both arguments.
theorem galois_coinsertion_join_lte_join[A: PartialOrder, B: JoinSemilattice](
    lower: A -> B,
    upper: B -> A,
    a: A,
    b: A,
    c: A,
    d: A
) {
    is_galois_coinsertion(lower, upper) and a <= b and c <= d implies
    galois_coinsertion_join(lower, upper, a, c) <= galois_coinsertion_join(lower, upper, b, d)
} by {
    if is_galois_coinsertion(lower, upper) and a <= b and c <= d {
        galois_coinsertion_lte_join_left(lower, upper, b, d)
        lte_trans(a, b, galois_coinsertion_join(lower, upper, b, d))
        galois_coinsertion_lte_join_right(lower, upper, b, d)
        lte_trans(c, d, galois_coinsertion_join(lower, upper, b, d))
        c <= galois_coinsertion_join(lower, upper, b, d)
        galois_coinsertion_join_lte_of_bounds(
            lower, upper, a, c, galois_coinsertion_join(lower, upper, b, d)
        )
        galois_coinsertion_join(lower, upper, a, c) <= galois_coinsertion_join(lower, upper, b, d)
    }
}

/// The transported meet is monotone in the left argument.
theorem galois_coinsertion_meet_lte_meet_left[A: PartialOrder, B: MeetSemilattice](
    lower: A -> B,
    upper: B -> A,
    a: A,
    b: A,
    c: A
) {
    is_galois_coinsertion(lower, upper) and a <= b implies
    galois_coinsertion_meet(lower, upper, a, c) <= galois_coinsertion_meet(lower, upper, b, c)
} by {
    if is_galois_coinsertion(lower, upper) and a <= b {
        lte_refl(c)
        galois_coinsertion_meet_lte_meet(lower, upper, a, b, c, c)
        galois_coinsertion_meet(lower, upper, a, c) <= galois_coinsertion_meet(lower, upper, b, c)
    }
}

/// The transported meet is monotone in the right argument.
theorem galois_coinsertion_meet_lte_meet_right[A: PartialOrder, B: MeetSemilattice](
    lower: A -> B,
    upper: B -> A,
    a: A,
    b: A,
    c: A
) {
    is_galois_coinsertion(lower, upper) and b <= c implies
    galois_coinsertion_meet(lower, upper, a, b) <= galois_coinsertion_meet(lower, upper, a, c)
} by {
    if is_galois_coinsertion(lower, upper) and b <= c {
        lte_refl(a)
        galois_coinsertion_meet_lte_meet(lower, upper, a, a, b, c)
        galois_coinsertion_meet(lower, upper, a, b) <= galois_coinsertion_meet(lower, upper, a, c)
    }
}

/// The transported join is monotone in the left argument.
theorem galois_coinsertion_join_lte_join_left[A: PartialOrder, B: JoinSemilattice](
    lower: A -> B,
    upper: B -> A,
    a: A,
    b: A,
    c: A
) {
    is_galois_coinsertion(lower, upper) and a <= b implies
    galois_coinsertion_join(lower, upper, a, c) <= galois_coinsertion_join(lower, upper, b, c)
} by {
    if is_galois_coinsertion(lower, upper) and a <= b {
        lte_refl(c)
        galois_coinsertion_join_lte_join(lower, upper, a, b, c, c)
        galois_coinsertion_join(lower, upper, a, c) <= galois_coinsertion_join(lower, upper, b, c)
    }
}

/// The transported join is monotone in the right argument.
theorem galois_coinsertion_join_lte_join_right[A: PartialOrder, B: JoinSemilattice](
    lower: A -> B,
    upper: B -> A,
    a: A,
    b: A,
    c: A
) {
    is_galois_coinsertion(lower, upper) and b <= c implies
    galois_coinsertion_join(lower, upper, a, b) <= galois_coinsertion_join(lower, upper, a, c)
} by {
    if is_galois_coinsertion(lower, upper) and b <= c {
        lte_refl(a)
        galois_coinsertion_join_lte_join(lower, upper, a, a, b, c)
        galois_coinsertion_join(lower, upper, a, b) <= galois_coinsertion_join(lower, upper, a, c)
    }
}

/// The transported meet is below its left argument.
theorem galois_coinsertion_meet_le_left[A: PartialOrder, B: MeetSemilattice](
    lower: A -> B,
    upper: B -> A,
    x: A,
    y: A
) {
    is_galois_coinsertion(lower, upper) implies
    galois_coinsertion_meet(lower, upper, x, y) <= x
} by {
    if is_galois_coinsertion(lower, upper) {
        galois_coinsertion_meet_lte_left(lower, upper, x, y)
        galois_coinsertion_meet(lower, upper, x, y) <= x
    }
}

/// The transported meet is below its right argument.
theorem galois_coinsertion_meet_le_right[A: PartialOrder, B: MeetSemilattice](
    lower: A -> B,
    upper: B -> A,
    x: A,
    y: A
) {
    is_galois_coinsertion(lower, upper) implies
    galois_coinsertion_meet(lower, upper, x, y) <= y
} by {
    if is_galois_coinsertion(lower, upper) {
        galois_coinsertion_meet_lte_right(lower, upper, x, y)
        galois_coinsertion_meet(lower, upper, x, y) <= y
    }
}

/// The left argument is below the transported join.
theorem galois_coinsertion_le_join_left[A: PartialOrder, B: JoinSemilattice](
    lower: A -> B,
    upper: B -> A,
    x: A,
    y: A
) {
    is_galois_coinsertion(lower, upper) implies
    x <= galois_coinsertion_join(lower, upper, x, y)
} by {
    if is_galois_coinsertion(lower, upper) {
        galois_coinsertion_lte_join_left(lower, upper, x, y)
        x <= galois_coinsertion_join(lower, upper, x, y)
    }
}

/// The right argument is below the transported join.
theorem galois_coinsertion_le_join_right[A: PartialOrder, B: JoinSemilattice](
    lower: A -> B,
    upper: B -> A,
    x: A,
    y: A
) {
    is_galois_coinsertion(lower, upper) implies
    y <= galois_coinsertion_join(lower, upper, x, y)
} by {
    if is_galois_coinsertion(lower, upper) {
        galois_coinsertion_lte_join_right(lower, upper, x, y)
        y <= galois_coinsertion_join(lower, upper, x, y)
    }
}

/// True if a self-map lies above every element.
define is_extensive[A: PartialOrder](f: A -> A) -> Bool {
    forall(a: A) {
        a <= f(a)
    }
}

/// True if a self-map lies below every element.
define is_reductive[A: PartialOrder](f: A -> A) -> Bool {
    forall(a: A) {
        f(a) <= a
    }
}

/// True if applying a self-map twice is the same as applying it once.
define is_idempotent_map[A](f: A -> A) -> Bool {
    forall(a: A) {
        f(f(a)) = f(a)
    }
}

/// True if a map is a closure operator.
define is_closure_operator[A: PartialOrder](close: A -> A) -> Bool {
    is_monotone(close) and is_extensive(close) and is_idempotent_map(close)
}

/// True if a map is a kernel operator.
define is_kernel_operator[A: PartialOrder](kernel: A -> A) -> Bool {
    is_monotone(kernel) and is_reductive(kernel) and is_idempotent_map(kernel)
}

/// True if a self-map fixes an element.
define is_fixed_by[A](f: A -> A, a: A) -> Bool {
    f(a) = a
}

/// A closure operator is monotone.
theorem closure_operator_is_monotone[A: PartialOrder](close: A -> A) {
    is_closure_operator(close) implies is_monotone(close)
} by {
    if is_closure_operator(close) {
        is_closure_operator(close) =
            (is_monotone(close) and is_extensive(close) and is_idempotent_map(close))
        is_monotone(close)
    }
}

/// A closure operator is extensive.
theorem closure_operator_is_extensive[A: PartialOrder](close: A -> A) {
    is_closure_operator(close) implies is_extensive(close)
} by {
    if is_closure_operator(close) {
        is_closure_operator(close) =
            (is_monotone(close) and is_extensive(close) and is_idempotent_map(close))
        is_extensive(close)
    }
}

/// A closure operator is idempotent.
theorem closure_operator_is_idempotent[A: PartialOrder](close: A -> A) {
    is_closure_operator(close) implies is_idempotent_map(close)
} by {
    if is_closure_operator(close) {
        is_idempotent_map(close)
    }
}

/// A closure operator lies above an element.
theorem closure_operator_extensive_at[A: PartialOrder](close: A -> A, a: A) {
    is_closure_operator(close) implies a <= close(a)
} by {
    if is_closure_operator(close) {
        closure_operator_is_extensive(close)
        a <= close(a)
    }
}

/// Applying a closure operator twice is the same as applying it once.
theorem closure_operator_idempotent_at[A: PartialOrder](close: A -> A, a: A) {
    is_closure_operator(close) implies close(close(a)) = close(a)
} by {
    if is_closure_operator(close) {
        closure_operator_is_idempotent(close)
        close(close(a)) = close(a)
    }
}

/// The value of a closure operator is fixed by that operator.
theorem closure_operator_image_fixed[A: PartialOrder](close: A -> A, a: A) {
    is_closure_operator(close) implies is_fixed_by(close, close(a))
} by {
    if is_closure_operator(close) {
        closure_operator_idempotent_at(close, a)
        is_fixed_by(close, close(a))
    }
}

/// The fixed elements of a closure operator are exactly its values.
theorem closure_operator_fixed_iff_image[A: PartialOrder](close: A -> A, c: A) {
    is_closure_operator(close) implies (is_fixed_by(close, c) iff exists(a: A) { close(a) = c })
} by {
    if is_closure_operator(close) {
        if is_fixed_by(close, c) {
            exists(a: A) { close(a) = c }
        }
        if exists(a: A) { close(a) = c } {
            let a: A satisfy {
                close(a) = c
            }
            closure_operator_image_fixed(close, a)
            is_fixed_by(close, c)
        }
    }
}

/// A fixed element above an element also lies above its closure.
theorem closure_operator_le_fixed_of_le[A: PartialOrder](close: A -> A, a: A, c: A) {
    is_closure_operator(close) and is_fixed_by(close, c) and a <= c implies close(a) <= c
} by {
    if is_closure_operator(close) and is_fixed_by(close, c) and a <= c {
        closure_operator_is_monotone(close)
        monotone_step(close, a, c)
        close(c) = c
        close(a) <= c
    }
}

/// Below a fixed element, comparing with the closure is the same as comparing with the element.
theorem closure_operator_le_fixed_iff_le[A: PartialOrder](close: A -> A, a: A, c: A) {
    is_closure_operator(close) and is_fixed_by(close, c) implies (close(a) <= c = (a <= c))
} by {
    if is_closure_operator(close) and is_fixed_by(close, c) {
        if close(a) <= c {
            closure_operator_extensive_at(close, a)
            lte_trans(a, close(a), c)
            a <= c
        }
        if a <= c {
            closure_operator_le_fixed_of_le(close, a, c)
            close(a) <= c
        }
        close(a) <= c = (a <= c)
    }
}

/// A kernel operator is monotone.
theorem kernel_operator_is_monotone[A: PartialOrder](kernel: A -> A) {
    is_kernel_operator(kernel) implies is_monotone(kernel)
} by {
    if is_kernel_operator(kernel) {
        is_kernel_operator(kernel) =
            (is_monotone(kernel) and is_reductive(kernel) and is_idempotent_map(kernel))
        is_monotone(kernel)
    }
}

/// A kernel operator is reductive.
theorem kernel_operator_is_reductive[A: PartialOrder](kernel: A -> A) {
    is_kernel_operator(kernel) implies is_reductive(kernel)
} by {
    if is_kernel_operator(kernel) {
        is_kernel_operator(kernel) =
            (is_monotone(kernel) and is_reductive(kernel) and is_idempotent_map(kernel))
        is_reductive(kernel)
    }
}

/// A kernel operator is idempotent.
theorem kernel_operator_is_idempotent[A: PartialOrder](kernel: A -> A) {
    is_kernel_operator(kernel) implies is_idempotent_map(kernel)
} by {
    if is_kernel_operator(kernel) {
        is_idempotent_map(kernel)
    }
}

/// A kernel operator lies below an element.
theorem kernel_operator_reductive_at[A: PartialOrder](kernel: A -> A, a: A) {
    is_kernel_operator(kernel) implies kernel(a) <= a
} by {
    if is_kernel_operator(kernel) {
        kernel_operator_is_reductive(kernel)
        kernel(a) <= a
    }
}

/// Applying a kernel operator twice is the same as applying it once.
theorem kernel_operator_idempotent_at[A: PartialOrder](kernel: A -> A, a: A) {
    is_kernel_operator(kernel) implies kernel(kernel(a)) = kernel(a)
} by {
    if is_kernel_operator(kernel) {
        kernel_operator_is_idempotent(kernel)
        kernel(kernel(a)) = kernel(a)
    }
}

/// The value of a kernel operator is fixed by that operator.
theorem kernel_operator_image_fixed[A: PartialOrder](kernel: A -> A, a: A) {
    is_kernel_operator(kernel) implies is_fixed_by(kernel, kernel(a))
} by {
    if is_kernel_operator(kernel) {
        kernel_operator_idempotent_at(kernel, a)
        is_fixed_by(kernel, kernel(a))
    }
}

/// The fixed elements of a kernel operator are exactly its values.
theorem kernel_operator_fixed_iff_image[A: PartialOrder](kernel: A -> A, k: A) {
    is_kernel_operator(kernel) implies (is_fixed_by(kernel, k) iff exists(a: A) { kernel(a) = k })
} by {
    if is_kernel_operator(kernel) {
        if is_fixed_by(kernel, k) {
            exists(a: A) { kernel(a) = k }
        }
        if exists(a: A) { kernel(a) = k } {
            let a: A satisfy {
                kernel(a) = k
            }
            kernel_operator_image_fixed(kernel, a)
            is_fixed_by(kernel, k)
        }
    }
}

/// A fixed element below an element also lies below its kernel.
theorem fixed_le_kernel_operator_of_le[A: PartialOrder](kernel: A -> A, k: A, a: A) {
    is_kernel_operator(kernel) and is_fixed_by(kernel, k) and k <= a implies k <= kernel(a)
} by {
    if is_kernel_operator(kernel) and is_fixed_by(kernel, k) and k <= a {
        kernel_operator_is_monotone(kernel)
        monotone_step(kernel, k, a)
        kernel(k) = k
        k <= kernel(a)
    }
}

/// Above a fixed element, comparing with the kernel is the same as comparing with the element.
theorem fixed_le_kernel_operator_iff_le[A: PartialOrder](kernel: A -> A, k: A, a: A) {
    is_kernel_operator(kernel) and is_fixed_by(kernel, k) implies (k <= kernel(a) = (k <= a))
} by {
    if is_kernel_operator(kernel) and is_fixed_by(kernel, k) {
        if k <= kernel(a) {
            kernel_operator_reductive_at(kernel, a)
            lte_trans(k, kernel(a), a)
            k <= a
        }
        if k <= a {
            fixed_le_kernel_operator_of_le(kernel, k, a)
            k <= kernel(a)
        }
        k <= kernel(a) = (k <= a)
    }
}

/// The closure induced by a Galois connection is a closure operator.
theorem galois_closure_is_closure_operator[A: PartialOrder, B: PartialOrder](
    lower: A -> B,
    upper: B -> A
) {
    is_galois_connection(lower, upper) implies
    is_closure_operator(galois_closure(lower, upper))
} by {
    if is_galois_connection(lower, upper) {
        galois_closure_monotone(lower, upper)
        forall(a: A) {
            galois_closure_extensive(lower, upper, a)
            a <= galois_closure(lower, upper, a)
        }
        is_extensive(galois_closure(lower, upper))
        forall(a: A) {
            galois_closure_idempotent(lower, upper, a)
            galois_closure(lower, upper, galois_closure(lower, upper, a)) =
            galois_closure(lower, upper, a)
        }
        is_idempotent_map(galois_closure(lower, upper))
        is_closure_operator(galois_closure(lower, upper))
    }
}

/// The kernel induced by a Galois connection is a kernel operator.
theorem galois_kernel_is_kernel_operator[A: PartialOrder, B: PartialOrder](
    lower: A -> B,
    upper: B -> A
) {
    is_galois_connection(lower, upper) implies
    is_kernel_operator(galois_kernel(lower, upper))
} by {
    if is_galois_connection(lower, upper) {
        galois_kernel_monotone(lower, upper)
        forall(b: B) {
            galois_kernel_reductive(lower, upper, b)
            galois_kernel(lower, upper, b) <= b
        }
        is_reductive(galois_kernel(lower, upper))
        forall(b: B) {
            galois_kernel_idempotent(lower, upper, b)
            galois_kernel(lower, upper, galois_kernel(lower, upper, b)) =
            galois_kernel(lower, upper, b)
        }
        is_idempotent_map(galois_kernel(lower, upper))
        is_kernel_operator(galois_kernel(lower, upper))
    }
}

/// The Galois closure of any element is fixed by the Galois closure.
theorem galois_closure_image_fixed[A: PartialOrder, B: PartialOrder](
    lower: A -> B,
    upper: B -> A,
    a: A
) {
    is_galois_connection(lower, upper) implies
    is_fixed_by(galois_closure(lower, upper), galois_closure(lower, upper, a))
} by {
    if is_galois_connection(lower, upper) {
        galois_closure_is_closure_operator(lower, upper)
        closure_operator_image_fixed(galois_closure(lower, upper), a)
        is_fixed_by(galois_closure(lower, upper), galois_closure(lower, upper, a))
    }
}

/// The Galois kernel of any element is fixed by the Galois kernel.
theorem galois_kernel_image_fixed[A: PartialOrder, B: PartialOrder](
    lower: A -> B,
    upper: B -> A,
    b: B
) {
    is_galois_connection(lower, upper) implies
    is_fixed_by(galois_kernel(lower, upper), galois_kernel(lower, upper, b))
} by {
    if is_galois_connection(lower, upper) {
        galois_kernel_is_kernel_operator(lower, upper)
        kernel_operator_image_fixed(galois_kernel(lower, upper), b)
        is_fixed_by(galois_kernel(lower, upper), galois_kernel(lower, upper, b))
    }
}

/// Every element in the upper adjoint image is fixed by the Galois closure.
theorem galois_closure_upper_fixed[A: PartialOrder, B: PartialOrder](
    lower: A -> B,
    upper: B -> A,
    b: B
) {
    is_galois_connection(lower, upper) implies
    is_fixed_by(galois_closure(lower, upper), upper(b))
} by {
    if is_galois_connection(lower, upper) {
        galois_closure_at(lower, upper, upper(b))
        galois_connection_counit(lower, upper, b)
        galois_connection_upper_monotone(lower, upper)
        monotone_step(upper, lower(upper(b)), b)
        galois_connection_unit(lower, upper, upper(b))
        upper(b) <= upper(lower(upper(b)))
        lte_antisymm(upper(lower(upper(b))), upper(b))
        upper(lower(upper(b))) = upper(b)
        is_fixed_by(galois_closure(lower, upper), upper(b))
    }
}

/// Every element in the lower adjoint image is fixed by the Galois kernel.
theorem galois_kernel_lower_fixed[A: PartialOrder, B: PartialOrder](
    lower: A -> B,
    upper: B -> A,
    a: A
) {
    is_galois_connection(lower, upper) implies
    is_fixed_by(galois_kernel(lower, upper), lower(a))
} by {
    if is_galois_connection(lower, upper) {
        galois_kernel_at(lower, upper, lower(a))
        galois_connection_unit(lower, upper, a)
        galois_connection_lower_monotone(lower, upper)
        monotone_step(lower, a, upper(lower(a)))
        galois_connection_counit(lower, upper, lower(a))
        lower(upper(lower(a))) <= lower(a)
        lte_antisymm(lower(upper(lower(a))), lower(a))
        lower(upper(lower(a))) = lower(a)
        is_fixed_by(galois_kernel(lower, upper), lower(a))
    }
}

/// A closed element has a preimage under the upper adjoint.
theorem galois_closure_fixed_has_upper_preimage[A: PartialOrder, B: PartialOrder](
    lower: A -> B,
    upper: B -> A,
    a: A
) {
    is_galois_connection(lower, upper) and is_fixed_by(galois_closure(lower, upper), a)
    implies exists(b: B) { a = upper(b) }
} by {
    if is_galois_connection(lower, upper) and is_fixed_by(galois_closure(lower, upper), a) {
        is_fixed_by(galois_closure(lower, upper), a) =
            (galois_closure(lower, upper, a) = a)
        galois_closure_at(lower, upper, a)
        exists(b: B) { a = upper(b) }
    }
}

/// An element in the upper adjoint image is closed.
theorem galois_closure_fixed_of_upper_image[A: PartialOrder, B: PartialOrder](
    lower: A -> B,
    upper: B -> A,
    a: A
) {
    is_galois_connection(lower, upper) and exists(b: B) { a = upper(b) } implies
    is_fixed_by(galois_closure(lower, upper), a)
} by {
    if is_galois_connection(lower, upper) and exists(b: B) { a = upper(b) } {
        let b: B satisfy {
            a = upper(b)
        }
        galois_closure_upper_fixed(lower, upper, b)
        is_fixed_by(galois_closure(lower, upper), a)
    }
}

/// The fixed elements of the Galois closure are exactly the upper adjoint image.
theorem galois_closure_fixed_iff_upper_image[A: PartialOrder, B: PartialOrder](
    lower: A -> B,
    upper: B -> A,
    a: A
) {
    is_galois_connection(lower, upper) implies
    (is_fixed_by(galois_closure(lower, upper), a) iff exists(b: B) { a = upper(b) })
} by {
    if is_galois_connection(lower, upper) {
        if is_fixed_by(galois_closure(lower, upper), a) {
            galois_closure_fixed_has_upper_preimage(lower, upper, a)
            exists(b: B) { a = upper(b) }
        }
        if exists(b: B) { a = upper(b) } {
            galois_closure_fixed_of_upper_image(lower, upper, a)
            is_fixed_by(galois_closure(lower, upper), a)
        }
    }
}

/// A kernel-stable element has a preimage under the lower adjoint.
theorem galois_kernel_fixed_has_lower_preimage[A: PartialOrder, B: PartialOrder](
    lower: A -> B,
    upper: B -> A,
    b: B
) {
    is_galois_connection(lower, upper) and is_fixed_by(galois_kernel(lower, upper), b)
    implies exists(a: A) { b = lower(a) }
} by {
    if is_galois_connection(lower, upper) and is_fixed_by(galois_kernel(lower, upper), b) {
        is_fixed_by(galois_kernel(lower, upper), b) =
            (galois_kernel(lower, upper, b) = b)
        galois_kernel_at(lower, upper, b)
        exists(a: A) { b = lower(a) }
    }
}

/// An element in the lower adjoint image is kernel-stable.
theorem galois_kernel_fixed_of_lower_image[A: PartialOrder, B: PartialOrder](
    lower: A -> B,
    upper: B -> A,
    b: B
) {
    is_galois_connection(lower, upper) and exists(a: A) { b = lower(a) } implies
    is_fixed_by(galois_kernel(lower, upper), b)
} by {
    if is_galois_connection(lower, upper) and exists(a: A) { b = lower(a) } {
        let a: A satisfy {
            b = lower(a)
        }
        galois_kernel_lower_fixed(lower, upper, a)
        is_fixed_by(galois_kernel(lower, upper), b)
    }
}

/// The fixed elements of the Galois kernel are exactly the lower adjoint image.
theorem galois_kernel_fixed_iff_lower_image[A: PartialOrder, B: PartialOrder](
    lower: A -> B,
    upper: B -> A,
    b: B
) {
    is_galois_connection(lower, upper) implies
    (is_fixed_by(galois_kernel(lower, upper), b) iff exists(a: A) { b = lower(a) })
} by {
    if is_galois_connection(lower, upper) {
        if is_fixed_by(galois_kernel(lower, upper), b) {
            galois_kernel_fixed_has_lower_preimage(lower, upper, b)
            exists(a: A) { b = lower(a) }
        }
        if exists(a: A) { b = lower(a) } {
            galois_kernel_fixed_of_lower_image(lower, upper, b)
            is_fixed_by(galois_kernel(lower, upper), b)
        }
    }
}

/// A Galois insertion gives a closure operator on the lower side.
theorem galois_insertion_closure_is_closure_operator[A: PartialOrder, B: PartialOrder](
    lower: A -> B,
    upper: B -> A
) {
    is_galois_insertion(lower, upper) implies
    is_closure_operator(galois_closure(lower, upper))
} by {
    if is_galois_insertion(lower, upper) {
        galois_insertion_is_connection(lower, upper)
        galois_closure_is_closure_operator(lower, upper)
        is_closure_operator(galois_closure(lower, upper))
    }
}

/// A Galois coinsertion gives a kernel operator on the upper side.
theorem galois_coinsertion_kernel_is_kernel_operator[A: PartialOrder, B: PartialOrder](
    lower: A -> B,
    upper: B -> A
) {
    is_galois_coinsertion(lower, upper) implies
    is_kernel_operator(galois_kernel(lower, upper))
} by {
    if is_galois_coinsertion(lower, upper) {
        galois_coinsertion_is_connection(lower, upper)
        galois_kernel_is_kernel_operator(lower, upper)
        is_kernel_operator(galois_kernel(lower, upper))
    }
}

/// The upper adjoint of a Galois insertion is an order embedding.
theorem galois_insertion_upper_is_order_embedding[A: PartialOrder, B: PartialOrder](
    lower: A -> B,
    upper: B -> A
) {
    is_galois_insertion(lower, upper) implies is_order_embedding(upper)
} by {
    if is_galois_insertion(lower, upper) {
        galois_insertion_is_connection(lower, upper)
        is_galois_connection(lower, upper)
        galois_connection_upper_monotone(lower, upper)
        is_monotone(upper)
        galois_connection_lower_monotone(lower, upper)
        is_monotone(lower)
        forall(x: B, y: B) {
            if upper(x) <= upper(y) {
                monotone_step(lower, upper(x), upper(y))
                lower(upper(x)) <= lower(upper(y))
                galois_insertion_counit_eq(lower, upper, x)
                galois_insertion_counit_eq(lower, upper, y)
                x <= y
            }
            if x <= y {
                monotone_step(upper, x, y)
                upper(x) <= upper(y)
            }
            upper(x) <= upper(y) = (x <= y)
        }
        is_order_embedding(upper)
    }
}

/// The lower adjoint of a Galois insertion is an order-preserving surjection.
theorem galois_insertion_lower_is_order_surjection[A: PartialOrder, B: PartialOrder](
    lower: A -> B,
    upper: B -> A
) {
    is_galois_insertion(lower, upper) implies is_order_surjection(lower)
} by {
    if is_galois_insertion(lower, upper) {
        galois_insertion_is_connection(lower, upper)
        galois_connection_lower_monotone(lower, upper)
        forall(b: B) {
            exists(a: A) {
                a = upper(b) and lower(a) = b
            }
        }
        is_surjective_fn(lower)
        is_order_surjection(lower)
    }
}

/// The lower adjoint of a Galois coinsertion is an order embedding.
theorem galois_coinsertion_lower_is_order_embedding[A: PartialOrder, B: PartialOrder](
    lower: A -> B,
    upper: B -> A
) {
    is_galois_coinsertion(lower, upper) implies is_order_embedding(lower)
} by {
    if is_galois_coinsertion(lower, upper) {
        galois_coinsertion_is_connection(lower, upper)
        is_galois_connection(lower, upper)
        galois_connection_lower_monotone(lower, upper)
        is_monotone(lower)
        galois_connection_upper_monotone(lower, upper)
        is_monotone(upper)
        forall(x: A, y: A) {
            if lower(x) <= lower(y) {
                monotone_step(upper, lower(x), lower(y))
                upper(lower(x)) <= upper(lower(y))
                galois_coinsertion_unit_eq(lower, upper, x)
                galois_coinsertion_unit_eq(lower, upper, y)
                x <= y
            }
            if x <= y {
                monotone_step(lower, x, y)
                lower(x) <= lower(y)
            }
            lower(x) <= lower(y) = (x <= y)
        }
        is_order_embedding(lower)
    }
}

/// The upper adjoint of a Galois coinsertion is an order-preserving surjection.
theorem galois_coinsertion_upper_is_order_surjection[A: PartialOrder, B: PartialOrder](
    lower: A -> B,
    upper: B -> A
) {
    is_galois_coinsertion(lower, upper) implies is_order_surjection(upper)
} by {
    if is_galois_coinsertion(lower, upper) {
        galois_coinsertion_is_connection(lower, upper)
        galois_connection_upper_monotone(lower, upper)
        forall(a: A) {
            exists(b: B) {
                b = lower(a) and upper(b) = a
            }
        }
        is_surjective_fn(upper)
        is_order_surjection(upper)
    }
}

/// An element in the image of the upper adjoint of an insertion is closed.
theorem galois_insertion_closure_fixed_of_upper_image[A: PartialOrder, B: PartialOrder](
    lower: A -> B,
    upper: B -> A,
    a: A
) {
    is_galois_insertion(lower, upper) and exists(b: B) { a = upper(b) } implies
    is_fixed_by(galois_closure(lower, upper), a)
} by {
    if is_galois_insertion(lower, upper) and exists(b: B) { a = upper(b) } {
        let b: B satisfy {
            a = upper(b)
        }
        galois_insertion_counit_eq(lower, upper, b)
        galois_closure_at(lower, upper, upper(b))
        is_fixed_by(galois_closure(lower, upper), a)
    }
}

/// An element in the image of the lower adjoint of a coinsertion is kernel-stable.
theorem galois_coinsertion_kernel_fixed_of_lower_image[A: PartialOrder, B: PartialOrder](
    lower: A -> B,
    upper: B -> A,
    b: B
) {
    is_galois_coinsertion(lower, upper) and exists(a: A) { b = lower(a) } implies
    is_fixed_by(galois_kernel(lower, upper), b)
} by {
    if is_galois_coinsertion(lower, upper) and exists(a: A) { b = lower(a) } {
        let a: A satisfy {
            b = lower(a)
        }
        galois_coinsertion_unit_eq(lower, upper, a)
        galois_kernel_at(lower, upper, lower(a))
        is_fixed_by(galois_kernel(lower, upper), b)
    }
}

/// Below a closed element, comparing the closure is the same as comparing the element.
theorem galois_closure_le_iff_le_of_fixed[A: PartialOrder, B: PartialOrder](
    lower: A -> B,
    upper: B -> A,
    a: A,
    c: A
) {
    is_galois_connection(lower, upper) and is_fixed_by(galois_closure(lower, upper), c)
    implies (galois_closure(lower, upper, a) <= c = (a <= c))
} by {
    if is_galois_connection(lower, upper) and is_fixed_by(galois_closure(lower, upper), c) {
        if galois_closure(lower, upper, a) <= c {
            galois_closure_extensive(lower, upper, a)
            lte_trans(a, galois_closure(lower, upper, a), c)
            a <= c
        }
        if a <= c {
            galois_closure_monotone(lower, upper)
            monotone_step(galois_closure(lower, upper), a, c)
            galois_closure(lower, upper, c) = c
            galois_closure(lower, upper, a) <= c
        }
        galois_closure(lower, upper, a) <= c = (a <= c)
    }
}

/// Above a kernel-stable element, comparing with the kernel is the same as comparing with the element.
theorem le_galois_kernel_iff_le_of_fixed[A: PartialOrder, B: PartialOrder](
    lower: A -> B,
    upper: B -> A,
    k: B,
    b: B
) {
    is_galois_connection(lower, upper) and is_fixed_by(galois_kernel(lower, upper), k)
    implies (k <= galois_kernel(lower, upper, b) = (k <= b))
} by {
    if is_galois_connection(lower, upper) and is_fixed_by(galois_kernel(lower, upper), k) {
        if k <= galois_kernel(lower, upper, b) {
            galois_kernel_reductive(lower, upper, b)
            lte_trans(k, galois_kernel(lower, upper, b), b)
            k <= b
        }
        if k <= b {
            galois_kernel_monotone(lower, upper)
            monotone_step(galois_kernel(lower, upper), k, b)
            galois_kernel(lower, upper, k) = k
            k <= galois_kernel(lower, upper, b)
        }
        k <= galois_kernel(lower, upper, b) = (k <= b)
    }
}

/// The identity function pair is a Galois connection on any partial order.
theorem identity_galois_connection[A: PartialOrder] {
    is_galois_connection(identity_fn[A], identity_fn[A])
} by {
    forall(a: A, b: A) {
        identity_fn[A](a) <= b = (a <= identity_fn[A](b))
    }
}

/// The composition of two Galois connections is a Galois connection.
theorem compose_galois_connection[A: PartialOrder, B: PartialOrder, C: PartialOrder](
    lower1: A -> B,
    upper1: B -> A,
    lower2: B -> C,
    upper2: C -> B
) {
    is_galois_connection(lower1, upper1) and is_galois_connection(lower2, upper2) implies
        is_galois_connection(compose(lower2, lower1), compose(upper1, upper2))
} by {
    if is_galois_connection(lower1, upper1) and is_galois_connection(lower2, upper2) {
        forall(a: A, c: C) {
            galois_connection_at(lower2, upper2, lower1(a), c)
            galois_connection_at(lower1, upper1, a, upper2(c))
            compose(lower2, lower1)(a) <= c = (a <= compose(upper1, upper2)(c))
        }
    }
}

/// The identity function pair is a Galois insertion on any partial order.
theorem identity_galois_insertion[A: PartialOrder] {
    is_galois_insertion(identity_fn[A], identity_fn[A])
} by {
    identity_galois_connection[A]
    forall(b: A) {
        identity_fn[A](identity_fn[A](b)) = b
    }
}

/// The identity function pair is a Galois coinsertion on any partial order.
theorem identity_galois_coinsertion[A: PartialOrder] {
    is_galois_coinsertion(identity_fn[A], identity_fn[A])
} by {
    identity_galois_connection[A]
    forall(a: A) {
        identity_fn[A](identity_fn[A](a)) = a
    }
}

/// The composition of two Galois insertions is a Galois insertion.
theorem compose_galois_insertion[A: PartialOrder, B: PartialOrder, C: PartialOrder](
    lower1: A -> B,
    upper1: B -> A,
    lower2: B -> C,
    upper2: C -> B
) {
    is_galois_insertion(lower1, upper1) and is_galois_insertion(lower2, upper2) implies
        is_galois_insertion(compose(lower2, lower1), compose(upper1, upper2))
} by {
    if is_galois_insertion(lower1, upper1) and is_galois_insertion(lower2, upper2) {
        galois_insertion_is_connection(lower1, upper1)
        galois_insertion_is_connection(lower2, upper2)
        compose_galois_connection(lower1, upper1, lower2, upper2)
        is_galois_connection(compose(lower2, lower1), compose(upper1, upper2))
        forall(b: C) {
            galois_insertion_counit_eq(lower1, upper1, upper2(b))
            galois_insertion_counit_eq(lower2, upper2, b)
            compose(lower2, lower1)(compose(upper1, upper2)(b)) = b
        }
        is_galois_insertion(compose(lower2, lower1), compose(upper1, upper2))
    }
}

/// The composition of two Galois coinsertions is a Galois coinsertion.
theorem compose_galois_coinsertion[A: PartialOrder, B: PartialOrder, C: PartialOrder](
    lower1: A -> B,
    upper1: B -> A,
    lower2: B -> C,
    upper2: C -> B
) {
    is_galois_coinsertion(lower1, upper1) and is_galois_coinsertion(lower2, upper2) implies
        is_galois_coinsertion(compose(lower2, lower1), compose(upper1, upper2))
} by {
    if is_galois_coinsertion(lower1, upper1) and is_galois_coinsertion(lower2, upper2) {
        galois_coinsertion_is_connection(lower1, upper1)
        galois_coinsertion_is_connection(lower2, upper2)
        compose_galois_connection(lower1, upper1, lower2, upper2)
        is_galois_connection(compose(lower2, lower1), compose(upper1, upper2))
        forall(a: A) {
            galois_coinsertion_unit_eq(lower2, upper2, lower1(a))
            galois_coinsertion_unit_eq(lower1, upper1, a)
            compose(upper1, upper2)(compose(lower2, lower1)(a)) = a
        }
        is_galois_coinsertion(compose(lower2, lower1), compose(upper1, upper2))
    }
}

