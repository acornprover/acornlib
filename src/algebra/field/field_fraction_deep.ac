// Fraction deepening: further identities for the formal division operation
// `fdiv(a, b) = a * b.inverse` — powers of quotients, the inverse of a
// quotient, cross-multiplied addition, and cancellation of common factors.

from algebra.field.field import Field, inverse_inverse, inverse_dist, mul_inverse_right
from algebra.field.field_deep import fdiv, field_fdiv_mul_cancel, field_fdiv_scale, field_fdiv_add,
    field_fdiv_neg, field_neg_inverse, field_pow_inverse_nat
from algebra.ring.ring import Ring, mul_neg_left, mul_neg_right, mul_neg_neg, mul_zero_left
from algebra.ring.ring_axioms_deep import ring_neg_pow_two
from nat import Nat, pow_distrib_mul, pow_one, pow_add, pow_zero
numerals Nat

/// The inverse of a quotient is the quotient of the reversed parts.
theorem field_fdiv_inverse_rev[F: Field](a: F, b: F) {
    fdiv(a, b).inverse = fdiv(b, a)
} by {
    fdiv(a, b) = a * b.inverse
    fdiv(a, b).inverse = (a * b.inverse).inverse
    inverse_dist(a, b.inverse)
    (a * b.inverse).inverse = a.inverse * b.inverse.inverse
    inverse_inverse(b)
    b.inverse.inverse = b
    a.inverse * b.inverse.inverse = a.inverse * b
    a.inverse * b = b * a.inverse
    fdiv(b, a) = b * a.inverse
    fdiv(a, b).inverse = fdiv(b, a)
}

/// A quotient raised to a natural power is the quotient of the powers.
theorem field_fdiv_pow[F: Field](a: F, b: F, n: Nat) {
    fdiv(a, b).pow(n) = fdiv(a.pow(n), b.pow(n))
} by {
    fdiv(a, b) = a * b.inverse
    pow_distrib_mul(a, b.inverse, n)
    (a * b.inverse).pow(n) = a.pow(n) * b.inverse.pow(n)
    field_pow_inverse_nat(b, n)
    b.inverse.pow(n) = b.pow(n).inverse
    a.pow(n) * b.inverse.pow(n) = a.pow(n) * b.pow(n).inverse
    fdiv(a.pow(n), b.pow(n)) = a.pow(n) * b.pow(n).inverse
    fdiv(a, b).pow(n) = fdiv(a.pow(n), b.pow(n))
}

/// The square of a quotient is the quotient of the squares.
theorem field_fdiv_pow_two[F: Field](a: F, b: F) {
    fdiv(a, b).pow(2) = fdiv(a.pow(2), b.pow(2))
} by {
    field_fdiv_pow(a, b, Nat.2)
    fdiv(a, b).pow(2) = fdiv(a.pow(2), b.pow(2))
}

/// Cancelling a factor between the numerator and the divisor.
theorem field_fdiv_mul_cancel_left[F: Field](a: F, b: F) {
    a != F.0 implies fdiv(a * b, a) = b
} by {
    if a != F.0 {
        fdiv(a * b, a) = (a * b) * a.inverse
        (a * b) * a.inverse = (b * a) * a.inverse
        b * a = a * b
        (b * a) * a.inverse = b * (a * a.inverse)
        mul_inverse_right(a)
        a * a.inverse = F.1
        b * (a * a.inverse) = b * F.1
        b * F.1 = b
        fdiv(a * b, a) = b
    }
}

/// Cancelling a factor between the numerator and the divisor, right-ordered.
theorem field_fdiv_mul_cancel_right[F: Field](a: F, b: F) {
    a != F.0 implies fdiv(b * a, a) = b
} by {
    if a != F.0 {
        field_fdiv_mul_cancel_left(a, b)
        fdiv(a * b, a) = b
        a * b = b * a
        fdiv(a * b, a) = fdiv(b * a, a)
        fdiv(b * a, a) = b
    }
}

/// A quotient equals an element exactly when the numerator is the product.
theorem field_fdiv_eq_iff[F: Field](a: F, b: F, c: F) {
    b != F.0 implies ((fdiv(a, b) = c) = (a = c * b))
} by {
    if b != F.0 {
        if fdiv(a, b) = c {
            fdiv(a, b) * b = c * b
            field_fdiv_mul_cancel(a, b)
            fdiv(a, b) * b = a
            a = c * b
        }
        if a = c * b {
            fdiv(a, b) = fdiv(c * b, b)
            field_fdiv_mul_cancel_left(b, c)
            fdiv(b * c, b) = c
            b * c = c * b
            fdiv(c * b, b) = fdiv(b * c, b)
            fdiv(c * b, b) = c
            fdiv(a, b) = c
        }
        (fdiv(a, b) = c) = (a = c * b)
    }
}

/// A quotient is zero exactly when the numerator is zero.
theorem field_fdiv_eq_zero_iff[F: Field](a: F, b: F) {
    b != F.0 implies ((fdiv(a, b) = F.0) = (a = F.0))
} by {
    if b != F.0 {
        if fdiv(a, b) = F.0 {
            fdiv(a, b) * b = F.0 * b
            field_fdiv_mul_cancel(a, b)
            fdiv(a, b) * b = a
            F.0 * b = F.0
            a = F.0
        }
        if a = F.0 {
            fdiv(a, b) = F.0 * b.inverse
            mul_zero_left(b.inverse)
            F.0 * b.inverse = F.0
            fdiv(a, b) = F.0
        }
        (fdiv(a, b) = F.0) = (a = F.0)
    }
}

/// Adding quotients by cross multiplication.
theorem field_fdiv_add_cross[F: Field](a: F, b: F, c: F, d: F) {
    b != F.0 and d != F.0 implies fdiv(a, b) + fdiv(c, d) = fdiv(a * d + c * b, b * d)
} by {
    if b != F.0 and d != F.0 {
        field_fdiv_scale(a, b, d)
        d != F.0
        fdiv(a, b) = fdiv(a * d, b * d)
        field_fdiv_scale(c, d, b)
        b != F.0
        fdiv(c, d) = fdiv(c * b, d * b)
        d * b = b * d
        fdiv(c * b, d * b) = fdiv(c * b, b * d)
        fdiv(c, d) = fdiv(c * b, b * d)
        field_fdiv_add(fdiv(a * d, b * d), fdiv(c * b, b * d), b * d)
        fdiv(a * d, b * d) + fdiv(c * b, b * d) = fdiv(a * d + c * b, b * d)
        fdiv(a, b) + fdiv(c, d) = fdiv(a * d + c * b, b * d)
    }
}

/// The quotient of a negated divisor is the negation of the quotient.
theorem field_fdiv_neg_divisor[F: Field](a: F, b: F) {
    b != F.0 implies fdiv(a, -b) = -fdiv(a, b)
} by {
    if b != F.0 {
        fdiv(a, -b) = a * (-b).inverse
        field_neg_inverse(b)
        (-b).inverse = -(b.inverse)
        a * (-b).inverse = a * -(b.inverse)
        mul_neg_right(a, b.inverse)
        a * -(b.inverse) = -(a * b.inverse)
        fdiv(a, b) = a * b.inverse
        -(a * b.inverse) = -fdiv(a, b)
        fdiv(a, -b) = -fdiv(a, b)
    }
}

/// The quotient of two negations is the quotient.
theorem field_fdiv_neg_neg[F: Field](a: F, b: F) {
    b != F.0 implies fdiv(-a, -b) = fdiv(a, b)
} by {
    if b != F.0 {
        fdiv(-a, -b) = -fdiv(-a, b)
        field_fdiv_neg_divisor(-a, b)
        field_fdiv_neg(a, b)
        fdiv(-a, b) = -fdiv(a, b)
        -fdiv(-a, b) = -(-fdiv(a, b))
        inverse_inverse(fdiv(a, b))
        --fdiv(a, b) = fdiv(a, b)
        -(-fdiv(a, b)) = fdiv(a, b)
        fdiv(-a, -b) = fdiv(a, b)
    }
}
