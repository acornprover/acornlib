// Field-theory deepening: multiplicative cancellation, the zero-product
// property as an iff, inverse laws (injectivity of inversion, negation),
// powers, and a formal division operation `fdiv(a, b) = a * b.inverse`
// together with the standard fraction identities.

from algebra.field.field import Field, unique_inverse, inverse_inverse, inverse_one,
    mul_inverse_right, mul_inverse_left, inverse_dist, field_mul_eq_zero, field_mul_eq_zero_of_factor,
    mul_not_zero, inverse_not_zero, pow_not_zero
from algebra.ring.ring import Ring, mul_neg_left, mul_zero_left, mul_zero_right
from nat import Nat, pow_add, pow_one, pow_zero, alt_induction
numerals Nat

/// In a field, cancelling a nonzero left factor.
theorem field_mul_cancel_left[F: Field](a: F, b: F, c: F) {
    a != F.0 and a * b = a * c implies b = c
} by {
    if a != F.0 and a * b = a * c {
        mul_inverse_left(a)
        a.inverse * (a * b) = (a.inverse * a) * b
        a.inverse * a = F.1
        (a.inverse * a) * b = F.1 * b
        F.1 * b = b
        a.inverse * (a * b) = b
        a.inverse * (a * c) = (a.inverse * a) * c
        (a.inverse * a) * c = F.1 * c
        F.1 * c = c
        a.inverse * (a * c) = c
        a * b = a * c
        a.inverse * (a * b) = a.inverse * (a * c)
        b = c
    }
}

/// In a field, cancelling a nonzero right factor.
theorem field_mul_cancel_right[F: Field](a: F, b: F, c: F) {
    a != F.0 and b * a = c * a implies b = c
} by {
    if a != F.0 and b * a = c * a {
        mul_inverse_right(a)
        (b * a) * a.inverse = b * (a * a.inverse)
        a * a.inverse = F.1
        b * (a * a.inverse) = b * F.1
        b * F.1 = b
        (b * a) * a.inverse = b
        (c * a) * a.inverse = c * (a * a.inverse)
        c * (a * a.inverse) = c * F.1
        c * F.1 = c
        (c * a) * a.inverse = c
        b * a = c * a
        (b * a) * a.inverse = (c * a) * a.inverse
        b = c
    }
}

/// Multiplying by a nonzero element is injective, as an iff.
theorem field_mul_eq_mul_iff[F: Field](a: F, b: F, c: F) {
    a != F.0 implies (a * b = a * c) = (b = c)
} by {
    if a != F.0 {
        if a * b = a * c {
            field_mul_cancel_left(a, b, c)
            b = c
        }
        if b = c {
            a * b = a * c
        }
        (a * b = a * c) = (b = c)
    }
}

/// A product is zero exactly when one of the factors is zero.
theorem field_mul_eq_zero_iff[F: Field](a: F, b: F) {
    (a * b = F.0) = (a = F.0 or b = F.0)
} by {
    if a * b = F.0 {
        field_mul_eq_zero(a, b)
        a = F.0 or b = F.0
    }
    if a = F.0 or b = F.0 {
        field_mul_eq_zero_of_factor(a, b)
        a * b = F.0
    }
    (a * b = F.0) = (a = F.0 or b = F.0)
}

/// A square is zero exactly when the element is zero.
theorem field_sq_eq_zero_iff[F: Field](a: F) {
    (a * a = F.0) = (a = F.0)
} by {
    if a * a = F.0 {
        field_mul_eq_zero(a, a)
        if a = F.0 {
            a = F.0
        } else {
            a = F.0
        }
        a = F.0
    }
    if a = F.0 {
        a * a = F.0 * F.0
        mul_zero_left(F.0)
        F.0 * F.0 = F.0
        a * a = F.0
    }
    (a * a = F.0) = (a = F.0)
}

/// Inversion is injective: equal inverses have equal elements.
theorem field_inverse_eq_inverse_iff[F: Field](a: F, b: F) {
    (a.inverse = b.inverse) = (a = b)
} by {
    if a.inverse = b.inverse {
        inverse_inverse(a)
        a.inverse.inverse = a
        a.inverse.inverse = b.inverse.inverse
        inverse_inverse(b)
        b.inverse.inverse = b
        a = b
    }
    if a = b {
        a.inverse = b.inverse
    }
    (a.inverse = b.inverse) = (a = b)
}

/// An element is one exactly when its inverse is one.
theorem field_inverse_eq_one_iff[F: Field](a: F) {
    (a.inverse = F.1) = (a = F.1)
} by {
    if a.inverse = F.1 {
        inverse_inverse(a)
        a.inverse.inverse = a
        inverse_one[F]
        F.1.inverse = F.1
        a.inverse.inverse = F.1.inverse
        a = F.1
    }
    if a = F.1 {
        a.inverse = F.1.inverse
        inverse_one[F]
        F.1.inverse = F.1
        a.inverse = F.1
    }
    (a.inverse = F.1) = (a = F.1)
}

/// An element is zero exactly when its inverse is zero.
theorem field_inverse_eq_zero_iff[F: Field](a: F) {
    (a.inverse = F.0) = (a = F.0)
} by {
    if a.inverse = F.0 {
        inverse_inverse(a)
        a.inverse.inverse = a
        F.0.inverse = F.0
        a.inverse.inverse = F.0.inverse
        a = F.0
    }
    if a = F.0 {
        a.inverse = F.0.inverse
        F.0.inverse = F.0
        a.inverse = F.0
    }
    (a.inverse = F.0) = (a = F.0)
}

/// The inverse of a negated element is the negation of the inverse.
theorem field_neg_inverse[F: Field](a: F) {
    a != F.0 implies (-a).inverse = -(a.inverse)
} by {
    if a != F.0 {
        -a * a.inverse = -(a * a.inverse)
        mul_inverse_right(a)
        a * a.inverse = F.1
        -(a * a.inverse) = -F.1
        -a * a.inverse = -F.1
        (-a).inverse = -(a.inverse)
    }
}

/// The natural power of an inverse is the inverse of the power.
theorem field_pow_inverse_nat[F: Field](a: F, n: Nat) {
    a.inverse.pow(n) = a.pow(n).inverse
} by {
    define p(k: Nat) -> Bool {
        a.inverse.pow(k) = a.pow(k).inverse
    }
    pow_zero(a.inverse)
    a.inverse.pow(Nat.0) = F.1
    pow_zero(a)
    a.pow(Nat.0) = F.1
    inverse_one[F]
    F.1.inverse = F.1
    a.pow(Nat.0).inverse = F.1
    p(Nat.0)
    forall(k: Nat) {
        if p(k) {
            pow_add(a.inverse, k, Nat.1)
            a.inverse.pow(k) * a.inverse.pow(Nat.1) = a.inverse.pow(k + Nat.1)
            pow_one(a.inverse)
            a.inverse.pow(Nat.1) = a.inverse
            pow_add(a, k, Nat.1)
            a.pow(k) * a.pow(Nat.1) = a.pow(k + Nat.1)
            pow_one(a)
            a.pow(Nat.1) = a
            inverse_dist(a.pow(k), a)
            (a.pow(k) * a).inverse = a.pow(k).inverse * a.inverse
            a.inverse.pow(k) = a.pow(k).inverse
            a.inverse.pow(k) * a.inverse = a.pow(k).inverse * a.inverse
            a.pow(k + Nat.1) = a.pow(k) * a
            (a.pow(k + Nat.1)).inverse = a.pow(k).inverse * a.inverse
            k + Nat.1 = k.suc
            a.pow(k + Nat.1) = a.pow(k.suc)
            a.pow(k.suc).inverse = a.pow(k).inverse * a.inverse
            a.inverse.pow(k) * a.inverse = a.pow(k.suc).inverse
            a.inverse.pow(k + Nat.1) = a.inverse.pow(k.suc)
            a.inverse.pow(k.suc) = a.pow(k.suc).inverse
            p(k.suc)
        }
    }
    p(Nat.0) and forall(k: Nat) { p(k) implies p(k.suc) }
    alt_induction(p)
    forall(k: Nat) { p(k) }
}

/// A positive power is zero exactly when the element is zero.
theorem field_pow_eq_zero_iff[F: Field](a: F, n: Nat) {
    n != Nat.0 implies (a.pow(n) = F.0) = (a = F.0)
} by {
    if n != Nat.0 {
        let k: Nat satisfy { n = k.suc }
        define q(m: Nat) -> Bool {
            (a.pow(m.suc) = F.0) = (a = F.0)
        }
        a.pow(Nat.0.suc) = a * a.pow(Nat.0)
        pow_zero(a)
        a.pow(Nat.0) = F.1
        a.pow(Nat.1) = a * F.1
        a * F.1 = a
        a.pow(Nat.1) = a
        (a.pow(Nat.1) = F.0) = (a = F.0)
        q(Nat.0)
        forall(m: Nat) {
            if q(m) {
                a.pow(m.suc.suc) = a * a.pow(m.suc)
                q(m) = ((a.pow(m.suc) = F.0) = (a = F.0))
                if a.pow(m.suc.suc) = F.0 {
                    field_mul_eq_zero(a, a.pow(m.suc))
                    if a = F.0 {
                        a = F.0
                    } else {
                        a.pow(m.suc) = F.0
                        (a.pow(m.suc) = F.0) = (a = F.0)
                        a = F.0
                    }
                    a = F.0
                }
                if a = F.0 {
                    a.pow(m.suc.suc) = F.0 * a.pow(m.suc)
                    mul_zero_left(a.pow(m.suc))
                    F.0 * a.pow(m.suc) = F.0
                    a.pow(m.suc.suc) = F.0
                }
                (a.pow(m.suc.suc) = F.0) = (a = F.0)
                q(m.suc)
            }
        }
        q(Nat.0) and forall(m: Nat) { q(m) implies q(m.suc) }
        alt_induction(q)
        forall(m: Nat) { q(m) }
        q(k)
        (a.pow(k.suc) = F.0) = (a = F.0)
        (a.pow(n) = F.0) = (a = F.0)
    }
}

/// The quotient of a field element by a divisor: a * b.inverse.
define fdiv[F: Field](a: F, b: F) -> F {
    a * b.inverse
}

/// A quotient multiplied by its divisor recovers the numerator.
theorem field_fdiv_mul_cancel[F: Field](a: F, b: F) {
    b != F.0 implies fdiv(a, b) * b = a
} by {
    if b != F.0 {
        fdiv(a, b) = a * b.inverse
        fdiv(a, b) * b = (a * b.inverse) * b
        (a * b.inverse) * b = a * (b.inverse * b)
        mul_inverse_left(b)
        b.inverse * b = F.1
        a * (b.inverse * b) = a * F.1
        a * F.1 = a
        fdiv(a, b) * b = a
    }
}

/// A divisor multiplied by its quotient recovers the numerator.
theorem field_fdiv_cancel_mul[F: Field](a: F, b: F) {
    b != F.0 implies b * fdiv(a, b) = a
} by {
    if b != F.0 {
        fdiv(a, b) = a * b.inverse
        b * fdiv(a, b) = b * (a * b.inverse)
        b * (a * b.inverse) = (b * a) * b.inverse
        b * a = a * b
        (b * a) * b.inverse = (a * b) * b.inverse
        (a * b) * b.inverse = a * (b * b.inverse)
        mul_inverse_right(b)
        b * b.inverse = F.1
        a * (b * b.inverse) = a * F.1
        a * F.1 = a
        b * fdiv(a, b) = a
    }
}

/// Dividing by one leaves an element unchanged.
theorem field_fdiv_one[F: Field](a: F) {
    fdiv(a, F.1) = a
} by {
    fdiv(a, F.1) = a * F.1.inverse
    inverse_one[F]
    F.1.inverse = F.1
    a * F.1.inverse = a * F.1
    a * F.1 = a
    fdiv(a, F.1) = a
}

/// Dividing a nonzero element by itself gives one.
theorem field_fdiv_self[F: Field](a: F) {
    a != F.0 implies fdiv(a, a) = F.1
} by {
    if a != F.0 {
        fdiv(a, a) = a * a.inverse
        mul_inverse_right(a)
        a * a.inverse = F.1
        fdiv(a, a) = F.1
    }
}

/// Dividing zero by anything gives zero.
theorem field_fdiv_zero[F: Field](a: F) {
    fdiv(F.0, a) = F.0
} by {
    fdiv(F.0, a) = F.0 * a.inverse
    mul_zero_left(a.inverse)
    F.0 * a.inverse = F.0
    fdiv(F.0, a) = F.0
}

/// Dividing one by an element gives its inverse.
theorem field_fdiv_inverse[F: Field](a: F) {
    fdiv(F.1, a) = a.inverse
} by {
    fdiv(F.1, a) = F.1 * a.inverse
    F.1 * a.inverse = a.inverse
    fdiv(F.1, a) = a.inverse
}

/// Adding quotients over a common divisor.
theorem field_fdiv_add[F: Field](a: F, b: F, c: F) {
    fdiv(a, c) + fdiv(b, c) = fdiv(a + b, c)
} by {
    fdiv(a, c) = a * c.inverse
    fdiv(b, c) = b * c.inverse
    fdiv(a, c) + fdiv(b, c) = a * c.inverse + b * c.inverse
    a * c.inverse + b * c.inverse = (a + b) * c.inverse
    fdiv(a + b, c) = (a + b) * c.inverse
    fdiv(a, c) + fdiv(b, c) = fdiv(a + b, c)
}

/// Subtracting quotients over a common divisor.
theorem field_fdiv_sub[F: Field](a: F, b: F, c: F) {
    fdiv(a, c) - fdiv(b, c) = fdiv(a - b, c)
} by {
    fdiv(b, c) = b * c.inverse
    fdiv(a, c) - fdiv(b, c) = a * c.inverse - b * c.inverse
    a * c.inverse - b * c.inverse = (a - b) * c.inverse
    fdiv(a - b, c) = (a - b) * c.inverse
    fdiv(a, c) - fdiv(b, c) = fdiv(a - b, c)
}

/// Multiplying quotients: a/b * c/d = (a*c)/(b*d).
theorem field_fdiv_mul[F: Field](a: F, b: F, c: F, d: F) {
    fdiv(a, b) * fdiv(c, d) = fdiv(a * c, b * d)
} by {
    fdiv(a, b) = a * b.inverse
    fdiv(c, d) = c * d.inverse
    fdiv(a, b) * fdiv(c, d) = (a * b.inverse) * (c * d.inverse)
    (a * b.inverse) * (c * d.inverse) = ((a * b.inverse) * c) * d.inverse
    (a * b.inverse) * c = a * (b.inverse * c)
    b.inverse * c = c * b.inverse
    a * (b.inverse * c) = a * (c * b.inverse)
    a * (c * b.inverse) = (a * c) * b.inverse
    ((a * b.inverse) * c) * d.inverse = (a * c) * b.inverse * d.inverse
    inverse_dist(b, d)
    (b * d).inverse = b.inverse * d.inverse
    (a * c) * (b.inverse * d.inverse) = (a * c) * (b * d).inverse
    fdiv(a * c, b * d) = (a * c) * (b * d).inverse
    (a * b.inverse) * (c * d.inverse) = (a * c) * (b * d).inverse
    fdiv(a, b) * fdiv(c, d) = fdiv(a * c, b * d)
}

/// The quotient of a negated numerator is the negation of the quotient.
theorem field_fdiv_neg[F: Field](a: F, b: F) {
    fdiv(-a, b) = -fdiv(a, b)
} by {
    fdiv(-a, b) = -a * b.inverse
    mul_neg_left(a, b.inverse)
    -a * b.inverse = -(a * b.inverse)
    fdiv(a, b) = a * b.inverse
    -(a * b.inverse) = -fdiv(a, b)
    fdiv(-a, b) = -fdiv(a, b)
}

/// Adding a whole element to a quotient: a/b + c = (a + c*b)/b.
theorem field_fdiv_add_mul[F: Field](a: F, b: F, c: F) {
    b != F.0 implies fdiv(a, b) + c = fdiv(a + c * b, b)
} by {
    if b != F.0 {
        fdiv(a, b) = a * b.inverse
        fdiv(a, b) + c = a * b.inverse + c
        c = c * F.1
        c * F.1 = c * (b.inverse * b)
        mul_inverse_left(b)
        b.inverse * b = F.1
        c * (b.inverse * b) = (c * b.inverse) * b
        c * b.inverse = b.inverse * c
        (c * b.inverse) * b = (b.inverse * c) * b
        (b.inverse * c) * b = b.inverse * (c * b)
        a * b.inverse + c = a * b.inverse + (c * b.inverse) * b
        a * b.inverse + (c * b.inverse) * b = a * b.inverse + b.inverse * (c * b)
        a * b.inverse + b.inverse * (c * b) = (a + c * b) * b.inverse
        fdiv(a + c * b, b) = (a + c * b) * b.inverse
        fdiv(a, b) + c = fdiv(a + c * b, b)
    }
}

/// Scaling both parts of a quotient by a nonzero factor leaves it unchanged.
theorem field_fdiv_scale[F: Field](a: F, b: F, c: F) {
    c != F.0 implies fdiv(a, b) = fdiv(a * c, b * c)
} by {
    if c != F.0 {
        fdiv(a * c, b * c) = (a * c) * (b * c).inverse
        inverse_dist(b, c)
        (b * c).inverse = b.inverse * c.inverse
        (a * c) * (b * c).inverse = (a * c) * (b.inverse * c.inverse)
        (a * c) * (b.inverse * c.inverse) = ((a * c) * b.inverse) * c.inverse
        (a * c) * b.inverse = (c * a) * b.inverse
        c * a = a * c
        (c * a) * b.inverse = c * (a * b.inverse)
        ((a * c) * b.inverse) * c.inverse = (c * (a * b.inverse)) * c.inverse
        (c * (a * b.inverse)) * c.inverse = (a * b.inverse) * (c * c.inverse)
        mul_inverse_right(c)
        c * c.inverse = F.1
        (a * b.inverse) * (c * c.inverse) = (a * b.inverse) * F.1
        (a * b.inverse) * F.1 = a * b.inverse
        fdiv(a, b) = a * b.inverse
        fdiv(a * c, b * c) = a * b.inverse
        fdiv(a, b) = fdiv(a * c, b * c)
    }
}

/// Dividing by a quotient: a / (b / c) = (a * c) / b.
theorem field_fdiv_inv[F: Field](a: F, b: F, c: F) {
    fdiv(a, fdiv(b, c)) = fdiv(a * c, b)
} by {
    fdiv(b, c) = b * c.inverse
    fdiv(a, fdiv(b, c)) = a * (b * c.inverse).inverse
    inverse_dist(b, c.inverse)
    (b * c.inverse).inverse = b.inverse * c.inverse.inverse
    inverse_inverse(c)
    c.inverse.inverse = c
    b.inverse * c.inverse.inverse = b.inverse * c
    a * (b * c.inverse).inverse = a * (b.inverse * c)
    a * (b.inverse * c) = (a * c) * b.inverse
    fdiv(a * c, b) = (a * c) * b.inverse
    fdiv(a, fdiv(b, c)) = fdiv(a * c, b)
}
