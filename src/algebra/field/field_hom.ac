from algebra.add_group import right_cancel
from algebra.field.field import Field, unique_inverse, mul_not_zero
from data.basic.functions import identity_fn, compose, compose_assoc, compose_identity_left, compose_identity_right
from algebra.ring.ring import Ring
from algebra.ring.ring_hom import RingHom, is_ring_hom, ring_hom_one, ring_hom_zero, ring_hom_add, ring_hom_mul, ring_hom_neg, identity_fn_is_ring_hom, compose_is_ring_hom

/// True if a function between fields preserves both ring operations and the multiplicative identity.
define is_field_hom[F: Field, E: Field](f: F -> E) -> Bool {
    is_ring_hom(f)
}

/// A field homomorphism between two fields.
structure FieldHom[F: Field, E: Field] {
    /// The mapping for the homomorphism.
    hom: F -> E
} constraint {
    is_field_hom(hom)
}

/// Field homomorphism extensionality: two homomorphisms are equal when they agree on every input.
theorem field_hom_ext[F: Field, E: Field](f: FieldHom[F, E], g: FieldHom[F, E]) {
    (forall(a: F) { f.hom(a) = g.hom(a) }) implies f = g
} by {
    if forall(a: F) { f.hom(a) = g.hom(a) } {
        f.hom = g.hom
    }
}

attributes FieldHom[F: Field, E: Field] {
    /// Field homomorphism extensionality from pointwise equality of the homomorphism.
    let ext = field_hom_ext[F, E]
}

/// The ring homomorphism underlying a field homomorphism.
let field_hom_to_ring_hom[F: Field, E: Field](f: FieldHom[F, E]) -> result: RingHom[F, E] satisfy {
    RingHom.new(f.hom) = Option.some(result)
} by {
    is_ring_hom(f.hom)
}

/// The underlying function of the ring homomorphism associated to a field homomorphism.
theorem field_hom_to_ring_hom_hom[F: Field, E: Field](f: FieldHom[F, E]) {
    field_hom_to_ring_hom(f).hom = f.hom
}

/// A field homomorphism preserves the multiplicative identity.
theorem field_hom_one[F: Field, E: Field](f: FieldHom[F, E]) {
    f.hom(F.1) = E.1
} by {
    field_hom_to_ring_hom_hom(f)
    ring_hom_one(field_hom_to_ring_hom(f))
}

/// A field homomorphism preserves the additive identity.
theorem field_hom_zero[F: Field, E: Field](f: FieldHom[F, E]) {
    f.hom(F.0) = E.0
} by {
    field_hom_to_ring_hom_hom(f)
    ring_hom_zero(field_hom_to_ring_hom(f))
}

/// A field homomorphism preserves addition.
theorem field_hom_add[F: Field, E: Field](f: FieldHom[F, E], a: F, b: F) {
    f.hom(a + b) = f.hom(a) + f.hom(b)
} by {
    field_hom_to_ring_hom_hom(f)
    ring_hom_add(field_hom_to_ring_hom(f), a, b)
}

/// A field homomorphism preserves multiplication.
theorem field_hom_mul[F: Field, E: Field](f: FieldHom[F, E], a: F, b: F) {
    f.hom(a * b) = f.hom(a) * f.hom(b)
} by {
    field_hom_to_ring_hom_hom(f)
    ring_hom_mul(field_hom_to_ring_hom(f), a, b)
}

/// A field homomorphism preserves additive inverses.
theorem field_hom_neg[F: Field, E: Field](f: FieldHom[F, E], a: F) {
    f.hom(-a) = -f.hom(a)
} by {
    field_hom_to_ring_hom_hom(f)
    ring_hom_neg(field_hom_to_ring_hom(f), a)
}

/// A field homomorphism sends nonzero elements to nonzero elements.
theorem field_hom_nonzero[F: Field, E: Field](f: FieldHom[F, E], a: F) {
    a != F.0 implies f.hom(a) != E.0
} by {
    if a != F.0 {
        a * a.inverse = F.1
        field_hom_mul(f, a, a.inverse)
        f.hom(a * a.inverse) = f.hom(a) * f.hom(a.inverse)
        f.hom(a) * f.hom(a.inverse) = f.hom(F.1)
        field_hom_one(f)
        f.hom(a) * f.hom(a.inverse) = E.1
        if f.hom(a) = E.0 {
            E.0 * f.hom(a.inverse) = E.1
            E.0 = E.1
            false
        }
    }
}

/// A field homomorphism is injective.
theorem field_hom_injective[F: Field, E: Field](f: FieldHom[F, E], a: F, b: F) {
    f.hom(a) = f.hom(b) implies a = b
} by {
    if f.hom(a) = f.hom(b) {
        field_hom_add(f, a, -b)
        field_hom_neg(f, b)
        f.hom(a + -b) = f.hom(a) + -f.hom(b)
        f.hom(a + -b) = E.0
        if a + -b != F.0 {
            field_hom_nonzero(f, a + -b)
            false
        }
        right_cancel(-b, a, b)
        a = b
    }
}

/// The identity function on a field is a field homomorphism.
theorem identity_fn_is_field_hom[F: Field] {
    is_field_hom(identity_fn[F])
} by {
    identity_fn_is_ring_hom[F]
}

/// The composition of two field homomorphisms is a field homomorphism.
theorem compose_is_field_hom[F: Field, E: Field, K: Field](f: E -> K, g: F -> E) {
    is_field_hom(f) and is_field_hom(g) implies is_field_hom(compose(f, g))
} by {
    if is_field_hom(f) and is_field_hom(g) {
        is_ring_hom(g)
        compose_is_ring_hom(f, g)
        is_field_hom(compose(f, g))
    }
}

/// The identity field homomorphism on a field, which sends every element to itself.
let identity_field_hom[F: Field]: FieldHom[F, F] satisfy {
    FieldHom.new(identity_fn[F]) = Option.some(identity_field_hom)
}

/// The underlying function of the identity field homomorphism is the identity function.
theorem identity_field_hom_hom[F: Field] {
    identity_field_hom[F].hom = identity_fn[F]
}

/// The composition of two field homomorphisms.
let compose_field_hom[F: Field, E: Field, K: Field](f: FieldHom[E, K], g: FieldHom[F, E]) -> result: FieldHom[F, K] satisfy {
    FieldHom.new(compose(f.hom, g.hom)) = Option.some(result)
} by {
    compose_is_field_hom(f.hom, g.hom)
    is_field_hom(compose(f.hom, g.hom))
}

/// The underlying function of a composition of field homomorphisms is the composition of underlying functions.
theorem compose_field_hom_hom[F: Field, E: Field, K: Field](f: FieldHom[E, K], g: FieldHom[F, E]) {
    compose_field_hom(f, g).hom = compose(f.hom, g.hom)
}

/// Composition of field homomorphisms is associative.
theorem compose_field_hom_assoc[F: Field, E: Field, K: Field, L: Field](f: FieldHom[K, L], g: FieldHom[E, K], h: FieldHom[F, E]) {
    compose_field_hom(compose_field_hom(f, g), h) = compose_field_hom(f, compose_field_hom(g, h))
} by {
    let lhs = compose_field_hom(compose_field_hom(f, g), h)
    let rhs = compose_field_hom(f, compose_field_hom(g, h))
    lhs.hom = compose(compose_field_hom(f, g).hom, h.hom)
    compose_field_hom(f, g).hom = compose(f.hom, g.hom)
    lhs.hom = compose(compose(f.hom, g.hom), h.hom)
    rhs.hom = compose(f.hom, compose_field_hom(g, h).hom)
    compose_field_hom(g, h).hom = compose(g.hom, h.hom)
    rhs.hom = compose(f.hom, compose(g.hom, h.hom))
    compose_assoc(f.hom, g.hom, h.hom)
    lhs.hom = rhs.hom
}

/// Composing the identity field homomorphism on the left does nothing.
theorem compose_field_hom_identity_left[F: Field, E: Field](f: FieldHom[F, E]) {
    compose_field_hom(identity_field_hom[E], f) = f
} by {
    let lhs = compose_field_hom(identity_field_hom[E], f)
    lhs.hom = compose(identity_field_hom[E].hom, f.hom)
    identity_field_hom[E].hom = identity_fn[E]
    lhs.hom = compose(identity_fn[E], f.hom)
    compose_identity_left(f.hom)
    lhs.hom = f.hom
}

/// Composing the identity field homomorphism on the right does nothing.
theorem compose_field_hom_identity_right[F: Field, E: Field](f: FieldHom[F, E]) {
    compose_field_hom(f, identity_field_hom[F]) = f
} by {
    let lhs = compose_field_hom(f, identity_field_hom[F])
    lhs.hom = compose(f.hom, identity_field_hom[F].hom)
    identity_field_hom[F].hom = identity_fn[F]
    lhs.hom = compose(f.hom, identity_fn[F])
    compose_identity_right(f.hom)
    lhs.hom = f.hom
}

/// A field homomorphism preserves multiplicative inverses of nonzero elements.
theorem field_hom_inverse[F: Field, E: Field](f: FieldHom[F, E], a: F) {
    a != F.0 implies f.hom(a.inverse) = f.hom(a).inverse
} by {
    if a != F.0 {
        a * a.inverse = F.1
        field_hom_mul(f, a, a.inverse)
        field_hom_one(f)
        unique_inverse(f.hom(a), f.hom(a.inverse))
        f.hom(a.inverse) = f.hom(a).inverse
    }
}
