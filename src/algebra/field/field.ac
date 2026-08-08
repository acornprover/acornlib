from nat import Nat
from int import Int
from comm_ring import CommRing
from algebra.monoid.monoid import Monoid
from algebra.inverse import Inverse

/// A field is a commutative ring with multiplicative inverses for all non-zero elements.
typeclass F: Field extends CommRing, Inverse {
    /// We define the field inverse so that the inverse of zero is zero.
    /// It would be nice to instead express that an inverse is "not valid" or yields "no value"
    /// but it is not convenient to do so in the current type system.
    field_inverse_zero {
        F.0.inverse = F.0
    }

    /// The definition of "multiplicative inverse" in a field.
    field_inverse(f: F) {
        f != F.0 implies f * f.inverse = F.1
    }

    /// Zero and one are distinct elements.
    zero_and_one_are_distinct {
        F.0 != F.1
    }
}

attributes F: Field {
    /// Extends Monoid.pow to raise a field element to an integer.
    /// Similar to how we define Field.inverse this defines `F.0`.zpow(x) for negative x to be `0`.
    define zpow(self, exp: Int) -> F {
        match exp {
            Int.from_nat(n) {
                self.pow(n)
            }
            Int.neg_suc(k) {
                self.pow(k.suc).inverse
            }
        }
    }
}

theorem zero_pow_nonnegative[F: Field](n: Int) {
    n > Int.0 implies F.0.zpow(n) = F.0
} by {
    match n {
        Int.from_nat(k) {
            F.0.zpow(n) = F.0
        }
        Int.neg_suc(k) {
            n.is_negative
            n < Int.0
        }
    }
}

// inverses are unique
theorem unique_inverse[F: Field](a: F, b: F) {
    a * b = F.1 implies b = a.inverse
} by {
    if F.0 = F.1 {
    } else {
        a != F.0
        a * a.inverse * b = a.inverse
        F.1 * b = a.inverse
    }

}

theorem inverse_inverse[F: Field](a: F) {
    a.inverse.inverse = a
}

/// A nonzero field element has a right inverse.
theorem mul_inverse_right[F: Field](a: F) {
    a != F.0 implies a * a.inverse = F.1
} by {
    if a != F.0 {
        F.field_inverse(a)
    }
}

/// A nonzero field element has a left inverse.
theorem mul_inverse_left[F: Field](a: F) {
    a != F.0 implies a.inverse * a = F.1
} by {
    if a != F.0 {
        F.field_inverse(a)
        a.inverse * a = a * a.inverse
    }
}

/// The inverse of one is one.
theorem inverse_one[F: Field] {
    F.1.inverse = F.1
} by {
    unique_inverse[F](F.1, F.1)
}

/// Cancelling a nonzero factor against its inverse on the right.
theorem mul_inverse_cancel_right[F: Field](a: F, b: F) {
    a != F.0 implies b * a * a.inverse = b
} by {
    if a != F.0 {
        F.field_inverse(a)
        b * (a * a.inverse) = b * F.1
        b * a * a.inverse = b
    }
}

/// Cancelling a nonzero factor against its inverse on the left.
theorem mul_inverse_cancel_left[F: Field](a: F, b: F) {
    a != F.0 implies a.inverse * (a * b) = b
} by {
    if a != F.0 {
        mul_inverse_left[F](a)
        (a.inverse * a) * b = F.1 * b
        a.inverse * (a * b) = b
    }
}

// (a * b).inverse = a.inverse * b.inverse
theorem inverse_dist[F: Field](a: F, b: F) {
    (a * b).inverse = a.inverse * b.inverse
} by {
    if a = F.0 {
        (a * b).inverse = a.inverse * b.inverse
    } else {
        if b = F.0 {
        } else {
            (a * b) * (a.inverse * b.inverse) = F.1
            a.inverse * b.inverse = (a * b).inverse
        }
    }
}

// a^(-n) = (a.inverse)^n for n Nat and >=1
theorem pow_inverse_nat[F: Field](a: F, k: Nat) {
    // n = k + 1
    a.zpow(Int.neg_suc(k)) = a.inverse.zpow(Int.from_nat(k.suc))
} by {
    let f: Nat -> Bool = function(x: Nat) {
        a.zpow(Int.neg_suc(x)) = a.inverse.zpow(Int.from_nat(x.suc))
    }
    // base case
    a.inverse.pow(Nat.1) = a.inverse
    a.pow(Nat.1) = a
    a.inverse.zpow(Int.from_nat(Nat.0.suc)) = a.inverse.pow(Nat.0.suc)
    a.pow(Nat.0.suc).inverse = a.zpow(Int.neg_suc(Nat.0))
    f(Nat.0)

    // Inductive step
    forall(l: Nat) {
        if f(l) {
            (a.pow(l.suc) * a).inverse = a.pow(l.suc).inverse * a.inverse
            f(l.suc)
        }
    }
}

// a^(-n) = (a.inverse)^n
theorem pow_inverse[F: Field](a: F, n: Int) {
    a.zpow(-n) = a.inverse.zpow(n)
} by {
    match n {
        Int.from_nat(k) {
            if k = Nat.0 {
                -n = Int.0
                a.pow(Nat.0) = F.1
                a.inverse.pow(Nat.0) = F.1
                a.zpow(-n) = a.inverse.zpow(n)
            } else {
                let l: Nat satisfy { l.suc = k }
                a.zpow(Int.neg_suc(l)) = a.inverse.zpow(Int.from_nat(l.suc))
                a.zpow(-n) = a.inverse.zpow(n)
            }
        }
        Int.neg_suc(k) {
            a.zpow(Int.from_nat(k.suc)) = a.inverse.zpow(Int.neg_suc(k))
            a.zpow(-n) = a.inverse.zpow(n)
        }
    }
}

theorem pow_nat_times_inverse[F: Field](a: F, n: Nat) {
   a != F.0 implies a.pow(n.suc) * a.inverse = a.pow(n)
} by {
    let f: Nat -> Bool = function(x: Nat) {
        a.pow(x.suc) * a.inverse = a.pow(x)
    }

    // base case
    a.pow(Nat.0.suc) = a.pow(Nat.0) * a
    a.pow(Nat.0) = F.1
    a.pow(Nat.0.suc) = F.1 * a
    F.1 * a = a
    a * a.inverse = F.1
    a.pow(Nat.0.suc) * a.inverse = a * a.inverse
    a.pow(Nat.0.suc) * a.inverse = F.1
    a.pow(Nat.0.suc) * a.inverse = a.pow(Nat.0)
    f(Nat.0)

    // Induction step
    forall(l: Nat) {
        if f(l) {
            // f(l) means a.pow(l.suc) * a.inverse = a.pow(l)

            // a.pow(l.suc.suc) = a.pow(l.suc) * a

            // a.pow(l.suc.suc) * a.inverse = a.pow(l.suc) * a * a.inverse
            a.pow(l.suc.suc) * a.inverse = a.pow(l.suc) * a * a.inverse

            // a * a.inverse = F.1

            f(l.suc)
        }
    }
    f(n)
}

theorem pow_nat_sub_inverse[F: Field](a: F, n: Nat, m: Nat) {
    a != F.0 and m <= n implies a.pow(n) * a.inverse.pow(m) = a.pow(n - m)
} by {
    let f: Nat -> Bool = function(x: Nat) {
        x > n or a.pow(n) * a.inverse.pow(x) = a.pow(n - x)
    }

    // Goal: m <= n implies a.pow(n) * a.inverse.pow(m) = a.pow(n - m)
    // Equivalently: not (m <= n) or a.pow(n) * a.inverse.pow(m) = a.pow(n - m)
    // Which follows from f(m) since m <= n implies not (m > n)
    m > n = n < m
    not m <= n or not n < m
    not m <= n or not m > n

    // base case: f(Nat.0) means Nat.0 > n or a.pow(n) * a.inverse.pow(Nat.0) = a.pow(n - Nat.0)
    a.inverse.pow(Nat.0) = F.1
    a.pow(n) * F.1 = a.pow(n)
    a.pow(n) * a.inverse.pow(Nat.0) = a.pow(n)
    n - Nat.0 = n
    a.pow(n - Nat.0) = a.pow(n)
    a.pow(n) * a.inverse.pow(Nat.0) = a.pow(n - Nat.0)
    Nat.0 > n or a.pow(n) * a.inverse.pow(Nat.0) = a.pow(n - Nat.0)
    f(Nat.0)

    // Inductive step
    forall(l: Nat) {
        if f(l) {
            if l >= n {
                l.suc > n
                f(l.suc)
            } else {
                l.suc <= n
                a.pow(n) * a.inverse.pow(l.suc) = a.pow(n) * a.inverse.pow(l) * a.inverse
                // Use the induction hypothesis: l < n implies a.pow(n) * a.inverse.pow(l) = a.pow(n - l)
                l < n
                l > n or a.pow(n) * a.inverse.pow(l) = a.pow(n - l)
                a.pow(n) * a.inverse.pow(l) = a.pow(n - l)
                // Show n - l = (n - l.suc).suc
                n - l.suc + l.suc = n
                (n - l.suc).suc + l = n
                a.pow(n - l) * a.inverse = a.pow((n - l.suc).suc) * a.inverse
                a.pow((n - l.suc).suc) * a.inverse = a.pow(n - l.suc)
                f(l.suc)
            }
        }
    }
    f(m)
}

theorem pow_sub_inverse[F: Field](a: F, n: Int, m: Int) {
    a != F.0 and n >= Int.0 and m >= Int.0 implies a.zpow(n) * a.inverse.zpow(m) = a.zpow(n - m)
} by {
    let l: Nat satisfy {
        n = Int.from_nat(l)
    }
    let k: Nat satisfy {
        m = Int.from_nat(k)
    }
    if m <= n {
        a.zpow(n) * a.inverse.zpow(m) = a.pow(l) * a.inverse.pow(k)
        k <= l
        a.zpow(n) * a.inverse.zpow(m) = a.pow(l - k)
        a.zpow(n) * a.inverse.zpow(m) = a.zpow(n - m)
    } else {
        a.zpow(n) * a.inverse.zpow(m) = a.inverse.pow(k) * a.inverse.inverse.pow(l)
        l <= k
        a.inverse != F.0
        a.zpow(n) * a.inverse.zpow(m) = a.inverse.pow(k - l)
        a.zpow(n) * a.inverse.zpow(m) = a.zpow(n - m)
    }
}

theorem pow_add_nonnegative[F: Field](a: F, n: Int, m: Int) {
    n >= Int.0 and m >= Int.0 implies a.zpow(n) * a.zpow(m) = a.zpow(n + m)
} by {
    let l: Nat satisfy {
        n = Int.from_nat(l)
    }
    let k: Nat satisfy {
        m = Int.from_nat(k)
    }
    a.pow(l) * a.pow(k) = a.pow(l + k)
    Int.from_nat(l) + Int.from_nat(k) = Int.from_nat(l + k)
}

theorem pow_add[F: Field](a: F, n: Int, m: Int) {
    (n >= Int.0 and m >= Int.0) or a != F.0 implies a.zpow(n) * a.zpow(m) = a.zpow(n + m)
} by {
    if n >= Int.0 {
        if m >= Int.0 {
            a.zpow(n) * a.zpow(m) = a.zpow(n + m)
        } else {
            a.zpow(n) * a.zpow(m) = a.zpow(n) * a.inverse.zpow(-m)
            -m >= Int.0
            a.zpow(n) * a.inverse.zpow(-m) = a.zpow(n - (-m))
            n + m = n - -m
            a.zpow(n) * a.zpow(m) = a.zpow(n + m)
        }
    } else {
        -n >= Int.0
        if m >= Int.0 {
            a.zpow(m) * a.inverse.zpow(-n) = a.zpow(m - (-n))
            m + n = m - -n
            a.zpow(n) * a.zpow(m) = a.zpow(m - (-n))
            a.zpow(n) * a.zpow(m) = a.zpow(n + m)
        } else {
            -m >= Int.0
            a.zpow(n) * a.zpow(m) = a.inverse.zpow(-n) * a.inverse.zpow(-m)
            a.zpow(n) * a.zpow(m) = a.inverse.zpow(-n + (-m))
            a.zpow(n) * a.zpow(m) = a.zpow(n + m)
        }
    }
}

theorem zero_pow[F: Field](n: Int) {
    n != Int.0 implies F.0.zpow(n) = F.0
} by {
    if n > Int.0 {
        F.0.zpow(n) = F.0
    } else {
        n < Int.0
        n <= Int.0
        Int.0 - n = -n
        -n > Int.0
        F.0.zpow(-n) = F.0
        F.0.zpow(n) = F.0
    }
}

theorem pow_inverse2[F: Field](a: F, n: Int) {
    a.zpow(-n) = a.zpow(n).inverse
} by {
    if a = F.0 {
        if n = Int.0 {
            a.zpow(Int.0) = a.pow(Nat.0)
            a.pow(Nat.0) = F.1
            a.zpow(-n) = a.zpow(n).inverse
        } else {
            a.zpow(-n) = a.zpow(n).inverse
        }
    } else {
        a.zpow(n) * a.zpow(-n) = a.zpow(n + -n)
        a.zpow(Int.0) = a.zpow(Int.from_nat(Nat.0))
        a.pow(Nat.0) = F.1
        a.zpow(n) * a.zpow(-n) = F.1
        a.zpow(-n) = a.zpow(n).inverse
    }
}

theorem one_pow[F: Field](n: Int) {
    F.1.zpow(n) = F.1
} by {
    match n {
        Int.from_nat(k) {
            F.1.pow(k) = F.1
            F.1.zpow(n) = F.1
        }
        Int.neg_suc(k) {
            F.1 * F.1 = F.1
            F.1.inverse = F.1
            Int.from_nat(k.suc) = -n
            F.1.zpow(Int.from_nat(k.suc)) = F.1.pow(k.suc)
            F.1.zpow(n) = F.1.pow(k.suc)
            F.1.zpow(n) = F.1
        }
    }
}

theorem pow_pow_nonnegative[F: Field](a: F, n: Int, m: Int) {
    n >= Int.0 and m >= Int.0 implies a.zpow(n).zpow(m) = a.zpow(n * m)
} by {
     let l: Nat satisfy {
        n = Int.from_nat(l)
    }
    let k: Nat satisfy {
        m = Int.from_nat(k)
    }
    a.zpow(n * m) = a.pow(l * k)
}

// Proof that (a^n)^m = a ^ (n * m)
theorem pow_pow[F: Field](a: F, n: Int, m: Int) {
    a != F.0 or (n >= Int.0 and m >= Int.0) implies a.zpow(n).zpow(m) = a.zpow(n * m)
} by {
    if n >= Int.0 {
        if m >= Int.0 {
        } else {
            -m >= Int.0
            a.zpow(n).zpow(-m) = a.zpow(n * -m)
            a.zpow(n).zpow(m).inverse = a.zpow(n).zpow(-m)
            n * -m = -(n * m)
            a.zpow(n * m).inverse = a.zpow(-(n * m))
            a.zpow(n).zpow(m) = a.zpow(n * m)
        }
    } else {
        -n >= Int.0
        if m >= Int.0 {
            a.inverse.zpow(-n).zpow(m) = a.inverse.zpow(-n * m)
            --n = n
            --(n * m) = n * m
            a.zpow(n).zpow(m) = a.zpow(n * m)
        } else {
            -m >= Int.0
            // Use pow_inverse2: a.zpow(-n) = a.zpow(n).inverse
            a.zpow(-n) = a.zpow(n).inverse
            a.zpow(-m) = a.zpow(m).inverse

            // a.zpow(n).zpow(m) = a.zpow(n).zpow(-(-m)).inverse
            --m = m
            a.zpow(n).zpow(--m) = a.zpow(n).zpow(-(-m))
            a.zpow(n).zpow(-(-m)) = a.zpow(n).zpow(-m).inverse
            a.zpow(n).zpow(m) = a.zpow(n).zpow(-m).inverse

            // a.zpow(n).zpow(-m) = a.zpow(n * -m) using pow_pow
            // We have a != F.0, so pow_pow applies
            // Actually use the intermediate value: a.zpow(n) is computed from a.inverse
            // Since n < 0, a.zpow(n) = a.inverse.zpow(-n) where -n >= 0
            // So a.zpow(n).zpow(-m) = a.inverse.zpow(-n).zpow(-m)
            a.zpow(n) = a.inverse.zpow(-n)
            a.inverse != F.0
            a.inverse.zpow(-n).zpow(-m) = a.inverse.zpow((-n) * (-m))
            a.zpow(n).zpow(-m) = a.inverse.zpow((-n) * (-m))
            (-n) * (-m) = n * m
            // a.inverse.zpow(n * m) = a.zpow(-(n * m)).inverse = a.zpow(n * -m)

            // a.zpow(n * -m).inverse = a.zpow(-(n * -m)) using pow_inverse2
            a.zpow(n).zpow(m) = a.zpow(-(n * -m))
            a.zpow(n).zpow(m) = a.zpow(n * m)
        }
    }
}

theorem inverse_not_zero[F: Field](a: F) {
    a != F.0 implies a.inverse != F.0
}

theorem mul_not_zero[F: Field](a: F, b: F) {
    a != F.0 and b != F.0 implies a * b != F.0
} by {
    if a * b = F.0 {
        a * b * b.inverse = F.0
    }
}

/// In a field, the product of two nonzero elements is nonzero.
theorem field_mul_nonzero[F: Field](a: F, b: F) {
    a != F.0 and b != F.0 implies a * b != F.0
} by {
    if a != F.0 and b != F.0 {
        mul_not_zero(a, b)
    }
}

/// Squaring a nonzero field element is nonzero.
theorem field_square_nonzero[F: Field](a: F) {
    a != F.0 implies a * a != F.0
} by {
    if a != F.0 {
        field_mul_nonzero(a, a)
    }
}

/// A nonzero product has a nonzero left factor.
theorem field_left_nonzero_of_mul_nonzero[F: Field](a: F, b: F) {
    a * b != F.0 implies a != F.0
} by {
    if a * b != F.0 {
        if a = F.0 {
            F.0 * b = F.0
            false
        }
    }
}

/// A nonzero product has a nonzero right factor.
theorem field_right_nonzero_of_mul_nonzero[F: Field](a: F, b: F) {
    a * b != F.0 implies b != F.0
} by {
    if a * b != F.0 {
        if b = F.0 {
            a * F.0 = F.0
            false
        }
    }
}

/// In a field, a zero product has a zero factor.
theorem field_mul_eq_zero[F: Field](a: F, b: F) {
    a * b = F.0 implies a = F.0 or b = F.0
} by {
    if a * b = F.0 {
        if a = F.0 {
            a = F.0 or b = F.0
        } else {
            if b = F.0 {
                a = F.0 or b = F.0
            } else {
                mul_not_zero(a, b)
                false
            }
        }
    }
}

/// If either factor is zero, then the product is zero.
theorem field_mul_eq_zero_of_factor[F: Field](a: F, b: F) {
    a = F.0 or b = F.0 implies a * b = F.0
} by {
    if a = F.0 or b = F.0 {
        if a = F.0 {
            a * b = F.0
        } else {
            a * b = F.0
        }
    }
}

theorem pow_not_zero[F: Field](a: F, n: Nat) {
    a != F.0 implies a.pow(n) != F.0
} by {
    // Define a helper function for induction
    define f(x: Nat) -> Bool { a.pow(x) != F.0 }

    // Base case
    f(Nat.0)

    // Inductive step
    forall(x: Nat) {
        if f(x) {
            f(x.suc)
        }
    }
}
