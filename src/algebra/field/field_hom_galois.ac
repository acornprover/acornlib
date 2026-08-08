/// Unbundled Galois-connection and closure/kernel API for field-hom subfield image/preimage.

from algebra.field.field import Field
from algebra.field.field_hom import FieldHom
from algebra.subfield import Subfield, subfield_le, subfield_inter
from algebra.field.field_hom_image import field_hom_image, subfield_image, subfield_preimage,
    subfield_image_le, subfield_preimage_le, subfield_le_preimage_image,
    subfield_preimage_image_eq, subfield_image_preimage_le,
    subfield_image_le_field_hom_image, subfield_image_preimage_eq_inter,
    subfield_image_le_iff_le_preimage, subfield_image_preimage_eq_iff_le_image,
    subfield_image_preimage_image_eq

/// Unbundled Galois-connection predicate for subfield image/preimage along a field homomorphism.
define is_subfield_image_preimage_galois_connection[F: Field, E: Field](
    phi: FieldHom[F, E]
) -> Bool {
    forall(s: Subfield[F], t: Subfield[E]) {
        subfield_le(subfield_image(phi, s), t) = subfield_le(s, subfield_preimage(phi, t))
    }
}

/// Pointwise unbundled adjunction law for subfield image/preimage.
theorem subfield_image_preimage_galois_connection_at[F: Field, E: Field](
    phi: FieldHom[F, E], s: Subfield[F], t: Subfield[E]
) {
    subfield_le(subfield_image(phi, s), t) = subfield_le(s, subfield_preimage(phi, t))
} by {
    subfield_image_le_iff_le_preimage(phi, s, t)
}

/// Raw unbundled Galois connection law over `subfield_le`.
theorem field_hom_subfield_image_preimage_galois_connection_raw[F: Field, E: Field](
    phi: FieldHom[F, E]
) {
    forall(s: Subfield[F], t: Subfield[E]) {
        subfield_le(subfield_image(phi, s), t) = subfield_le(s, subfield_preimage(phi, t))
    }
} by {
    forall(s: Subfield[F], t: Subfield[E]) {
        subfield_image_preimage_galois_connection_at(phi, s, t)
    }
}

/// The domain-side image/preimage closure induced by a field homomorphism.
define subfield_image_preimage_closure[F: Field, E: Field](
    phi: FieldHom[F, E], s: Subfield[F]
) -> Subfield[F] {
    subfield_preimage(phi, subfield_image(phi, s))
}

/// The image/preimage closure is extensive on domain subfields.
theorem subfield_image_preimage_closure_extensive[F: Field, E: Field](
    phi: FieldHom[F, E], s: Subfield[F]
) {
    subfield_le(s, subfield_image_preimage_closure(phi, s))
} by {
    subfield_le_preimage_image(phi, s)
}

/// The image/preimage closure is monotone on domain subfields.
theorem subfield_image_preimage_closure_monotone[F: Field, E: Field](
    phi: FieldHom[F, E], s: Subfield[F], t: Subfield[F]
) {
    subfield_le(s, t) implies
        subfield_le(subfield_image_preimage_closure(phi, s), subfield_image_preimage_closure(phi, t))
} by {
    if subfield_le(s, t) {
        subfield_image_le(phi, s, t)
        subfield_preimage_le(phi, subfield_image(phi, s), subfield_image(phi, t))
        subfield_le(subfield_preimage(phi, subfield_image(phi, s)),
            subfield_preimage(phi, subfield_image(phi, t)))
        subfield_le(subfield_image_preimage_closure(phi, s), subfield_image_preimage_closure(phi, t))
    }
}

/// Image inclusion into `subfield_image(phi, t)` is equivalent to inclusion in the domain-side closure of `t`.
theorem subfield_le_image_preimage_closure_iff_image_le[F: Field, E: Field](
    phi: FieldHom[F, E], s: Subfield[F], t: Subfield[F]
) {
    subfield_le(s, subfield_image_preimage_closure(phi, t)) =
        subfield_le(subfield_image(phi, s), subfield_image(phi, t))
} by {
    subfield_image_le_iff_le_preimage(phi, s, subfield_image(phi, t))
}

/// For field homomorphisms, the domain-side closure is exactly the original subfield.
theorem subfield_image_preimage_closure_eq_self[F: Field, E: Field](
    phi: FieldHom[F, E], s: Subfield[F]
) {
    subfield_image_preimage_closure(phi, s) = s
} by {
    subfield_preimage_image_eq(phi, s)
}

/// The image of a domain subfield is unchanged by applying the domain-side closure first.
theorem subfield_image_of_preimage_closure_eq[F: Field, E: Field](
    phi: FieldHom[F, E], s: Subfield[F]
) {
    subfield_image(phi, subfield_image_preimage_closure(phi, s)) = subfield_image(phi, s)
} by {
    subfield_image_preimage_image_eq(phi, s)
}

/// The domain-side image/preimage closure is idempotent.
theorem subfield_image_preimage_closure_idempotent[F: Field, E: Field](
    phi: FieldHom[F, E], s: Subfield[F]
) {
    subfield_image_preimage_closure(phi, subfield_image_preimage_closure(phi, s)) =
        subfield_image_preimage_closure(phi, s)
} by {
    subfield_image_preimage_closure_eq_self(phi, subfield_image_preimage_closure(phi, s))
}

/// The codomain-side image/preimage kernel/interior induced by a field homomorphism.
define subfield_image_preimage_kernel[F: Field, E: Field](
    phi: FieldHom[F, E], t: Subfield[E]
) -> Subfield[E] {
    subfield_image(phi, subfield_preimage(phi, t))
}

/// The codomain-side kernel is reductive: it is contained in the original codomain subfield.
theorem subfield_image_preimage_kernel_le[F: Field, E: Field](
    phi: FieldHom[F, E], t: Subfield[E]
) {
    subfield_le(subfield_image_preimage_kernel(phi, t), t)
} by {
    subfield_image_preimage_le(phi, t)
}

/// The codomain-side kernel is always contained in the field-hom image.
theorem subfield_image_preimage_kernel_le_field_hom_image[F: Field, E: Field](
    phi: FieldHom[F, E], t: Subfield[E]
) {
    subfield_le(subfield_image_preimage_kernel(phi, t), field_hom_image(phi))
} by {
    subfield_image_le_field_hom_image(phi, subfield_preimage(phi, t))
}

/// The codomain-side kernel is exactly intersection with the image of the field homomorphism.
theorem subfield_image_preimage_kernel_eq_inter[F: Field, E: Field](
    phi: FieldHom[F, E], t: Subfield[E]
) {
    subfield_image_preimage_kernel(phi, t) = subfield_inter(t, field_hom_image(phi))
} by {
    subfield_image_preimage_eq_inter(phi, t)
}

/// Kernel inclusion into `t` is equivalent to preimage inclusion into `subfield_preimage(phi, t)`.
theorem subfield_image_preimage_kernel_le_iff_preimage_le[F: Field, E: Field](
    phi: FieldHom[F, E], s: Subfield[E], t: Subfield[E]
) {
    subfield_le(subfield_image_preimage_kernel(phi, s), t) =
        subfield_le(subfield_preimage(phi, s), subfield_preimage(phi, t))
} by {
    subfield_image_le_iff_le_preimage(phi, subfield_preimage(phi, s), t)
}

/// The codomain-side kernel is monotone on codomain subfields.
theorem subfield_image_preimage_kernel_monotone[F: Field, E: Field](
    phi: FieldHom[F, E], s: Subfield[E], t: Subfield[E]
) {
    subfield_le(s, t) implies
        subfield_le(subfield_image_preimage_kernel(phi, s), subfield_image_preimage_kernel(phi, t))
} by {
    if subfield_le(s, t) {
        subfield_preimage_le(phi, s, t)
        subfield_image_le(phi, subfield_preimage(phi, s), subfield_preimage(phi, t))
        subfield_le(subfield_image(phi, subfield_preimage(phi, s)),
            subfield_image(phi, subfield_preimage(phi, t)))
        subfield_le(subfield_image_preimage_kernel(phi, s), subfield_image_preimage_kernel(phi, t))
    }
}

/// The codomain-side kernel is idempotent.
theorem subfield_image_preimage_kernel_idempotent[F: Field, E: Field](
    phi: FieldHom[F, E], t: Subfield[E]
) {
    subfield_image_preimage_kernel(phi, subfield_image_preimage_kernel(phi, t)) =
        subfield_image_preimage_kernel(phi, t)
} by {
    subfield_image_preimage_image_eq(phi, subfield_preimage(phi, t))
}

/// Fixed points of the codomain-side kernel are exactly subfields contained in the field-hom image.
theorem subfield_image_preimage_kernel_fixed_iff_le_image[F: Field, E: Field](
    phi: FieldHom[F, E], t: Subfield[E]
) {
    (subfield_image_preimage_kernel(phi, t) = t) = subfield_le(t, field_hom_image(phi))
} by {
    subfield_image_preimage_eq_iff_le_image(phi, t)
}

/// If a codomain subfield lies in the field-hom image, then it is fixed by the kernel.
theorem subfield_image_preimage_kernel_eq_of_le_image[F: Field, E: Field](
    phi: FieldHom[F, E], t: Subfield[E]
) {
    subfield_le(t, field_hom_image(phi)) implies subfield_image_preimage_kernel(phi, t) = t
} by {
    if subfield_le(t, field_hom_image(phi)) {
        subfield_image_preimage_kernel_fixed_iff_le_image(phi, t)
        subfield_image_preimage_kernel(phi, t) = t
    }
}

/// A kernel-fixed codomain subfield is contained in the field-hom image.
theorem subfield_le_image_of_subfield_image_preimage_kernel_eq[F: Field, E: Field](
    phi: FieldHom[F, E], t: Subfield[E]
) {
    subfield_image_preimage_kernel(phi, t) = t implies subfield_le(t, field_hom_image(phi))
} by {
    if subfield_image_preimage_kernel(phi, t) = t {
        subfield_image_preimage_kernel_fixed_iff_le_image(phi, t)
        subfield_le(t, field_hom_image(phi))
    }
}
