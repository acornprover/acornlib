from algebra.field.field import Field
from algebra.ring.ideal import MaximalIdeal, is_ideal, is_ideal_absorb, ideal_subset, ideal_proper_constraint,
    is_maximal_ideal, is_maximal_ideal_from_constraints, is_maximal_ideal_implies_prime,
    is_prime_ideal, zero_ideal, zero_ideal_is_ideal

/// In a field, the zero ideal is proper.
theorem zero_ideal_is_proper[F: Field] {
    ideal_proper_constraint(zero_ideal[F])
} by {
    not zero_ideal[F](F.1)
}

/// In a field, the zero ideal does not contain one.
theorem zero_ideal_not_contains_one[F: Field] {
    not zero_ideal[F](F.1)
} by {
    zero_ideal_is_proper[F]
}

/// In a field, the zero ideal is a maximal ideal.
theorem zero_ideal_is_maximal[F: Field] {
    is_maximal_ideal(zero_ideal[F])
} by {
    zero_ideal_is_ideal[F]


    forall(other: F -> Bool) {
        if is_ideal(other) and ideal_subset(zero_ideal[F], other) and ideal_proper_constraint(other) {
            forall(x: F) {
                if other(x) {
                    if x != F.0 {
                        is_ideal_absorb(other, x.inverse, x)
                        x.inverse * x = F.1
                        other(F.1)
                        false
                    }
                    zero_ideal[F](x)
                }
            }
            ideal_subset(other, zero_ideal[F])
        }
    }
    is_maximal_ideal_from_constraints(zero_ideal[F])
}

/// In a field, the zero ideal is a prime ideal.
theorem zero_ideal_is_prime[F: Field] {
    is_prime_ideal(zero_ideal[F])
} by {
    zero_ideal_is_maximal[F]
    is_maximal_ideal_implies_prime(zero_ideal[F])
}

/// The zero ideal of a field as a bundled maximal ideal.
let field_zero_maximal_ideal[F: Field]: MaximalIdeal[F] satisfy {
    MaximalIdeal[F].new(zero_ideal[F]) = Option.some(field_zero_maximal_ideal)
}

/// Membership in the bundled maximal zero ideal is membership in the zero ideal predicate.
theorem field_zero_maximal_ideal_contains_eq[F: Field](x: F) {
    field_zero_maximal_ideal[F].contains(x) = zero_ideal[F](x)
}

/// In a field, the zero ideal is both maximal and prime.
theorem zero_ideal_is_maximal_and_prime[F: Field] {
    is_maximal_ideal(zero_ideal[F]) and is_prime_ideal(zero_ideal[F])
} by {
    zero_ideal_is_maximal[F]
    zero_ideal_is_prime[F]
}
