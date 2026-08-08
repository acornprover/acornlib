from algebra.field.field import Field
from algebra.field.field_hom import FieldHom, compose_field_hom, compose_field_hom_hom, identity_field_hom, identity_field_hom_hom, field_hom_zero, field_hom_one, field_hom_add, field_hom_mul, field_hom_neg, field_hom_inverse, field_hom_nonzero, field_hom_injective
from data.basic.functions import compose
from algebra.subfield import Subfield, subfield_constraint, subfield_zero_constraint, subfield_one_constraint, subfield_add_constraint, subfield_neg_constraint, subfield_mul_constraint, subfield_inverse_constraint, subfield_additive_constraint, subfield_multiplicative_constraint, subfield_ext, subfield_le, subfield_le_intro, subfield_le_trans, subfield_le_antisymm, subfield_contains_zero, subfield_contains_one, subfield_contains_add, subfield_contains_neg, subfield_contains_mul, subfield_contains_inverse, subfield_inter, subfield_inter_contains_eq, subfield_inter_eq_left_iff_le, subfield_inter_le_left, subfield_inter_le_right, subfield_inter_greatest

/// True if y lies in the image of the field homomorphism phi.
define field_hom_image_contains[F: Field, E: Field](phi: FieldHom[F, E], y: E) -> Bool {
    exists(x: F) { phi.hom(x) = y }
}

/// The image of a field homomorphism as a subfield of the codomain.
let field_hom_image[F: Field, E: Field](phi: FieldHom[F, E]) -> result: Subfield[E] satisfy {
    Subfield.new(field_hom_image_contains(phi)) = Option.some(result)
} by {
    let contains: E -> Bool = field_hom_image_contains(phi)
    field_hom_zero(phi)
    contains(E.0)
    subfield_zero_constraint(contains)
    field_hom_one(phi)
    contains(E.1)
    subfield_one_constraint(contains)
    forall(a: E, b: E) {
        if contains(a) and contains(b) {
            let xa: F satisfy { phi.hom(xa) = a }
            let xb: F satisfy { phi.hom(xb) = b }
            field_hom_add(phi, xa, xb)
            phi.hom(xa + xb) = a + b
            contains(a + b)
        }
    }
    subfield_add_constraint(contains)
    forall(a: E) {
        if contains(a) {
            let xa: F satisfy { phi.hom(xa) = a }
            field_hom_neg(phi, xa)
            contains(-a)
        }
    }
    subfield_neg_constraint(contains)
    subfield_additive_constraint(contains)
    forall(a: E, b: E) {
        if contains(a) and contains(b) {
            let xa: F satisfy { phi.hom(xa) = a }
            let xb: F satisfy { phi.hom(xb) = b }
            field_hom_mul(phi, xa, xb)
            phi.hom(xa * xb) = a * b
            contains(a * b)
        }
    }
    subfield_mul_constraint(contains)
    forall(a: E) {
        if contains(a) and a != E.0 {
            let xa: F satisfy { phi.hom(xa) = a }
            if xa = F.0 {
                field_hom_zero(phi)
                false
            }
            field_hom_inverse(phi, xa)
            phi.hom(xa.inverse) = a.inverse
            contains(a.inverse)
        }
    }
    subfield_inverse_constraint(contains)
    subfield_multiplicative_constraint(contains)
    subfield_constraint(contains)
}

/// The membership predicate of the image subfield is the image predicate.
theorem field_hom_image_contains_eq[F: Field, E: Field](phi: FieldHom[F, E]) {
    field_hom_image(phi).contains = field_hom_image_contains(phi)
}

/// Every value of a field homomorphism lies in its image subfield.
theorem field_hom_image_contains_hom[F: Field, E: Field](phi: FieldHom[F, E], x: F) {
    field_hom_image(phi).contains(phi.hom(x))
} by {
    field_hom_image_contains_eq(phi)
    field_hom_image_contains(phi, phi.hom(x))
}

/// True if every element of the codomain lies in the subfield (the whole-field membership predicate).
define top_subfield_contains[F: Field](x: F) -> Bool {
    true
}

/// The whole-field membership predicate satisfies the subfield constraint.
theorem top_subfield_contains_is_subfield[F: Field] {
    subfield_constraint(top_subfield_contains[F])
} by {
    let contains: F -> Bool = top_subfield_contains[F]
    subfield_zero_constraint(contains)
    subfield_one_constraint(contains)
    subfield_add_constraint(contains)
    subfield_neg_constraint(contains)
    subfield_mul_constraint(contains)
    subfield_inverse_constraint(contains)
    subfield_additive_constraint(contains)
    subfield_multiplicative_constraint(contains)
}

/// The whole field as a subfield of itself.
let top_subfield[F: Field]: Subfield[F] satisfy {
    Subfield.new(top_subfield_contains[F]) = Option.some(top_subfield)
}

/// The whole-field subfield contains every element.
theorem top_subfield_contains_all[F: Field](x: F) {
    top_subfield[F].contains(x)
} by {
    top_subfield[F].contains = top_subfield_contains[F]
}

/// The image of the identity field homomorphism is the whole field.
theorem field_hom_image_identity[F: Field] {
    field_hom_image(identity_field_hom[F]) = top_subfield[F]
} by {
    let lhs = field_hom_image(identity_field_hom[F])
    let rhs = top_subfield[F]
    forall(y: F) {
        identity_field_hom[F].hom(y) = y
        field_hom_image_contains_hom(identity_field_hom[F], y)
        top_subfield_contains_all(y)
        lhs.contains(y) = rhs.contains(y)
    }
    subfield_ext(lhs, rhs)
}

/// Every subfield is contained in the whole-field subfield.
theorem top_subfield_ge[F: Field](s: Subfield[F]) {
    subfield_le(s, top_subfield[F])
} by {
    forall(x: F) {
        if s.contains(x) {
            top_subfield_contains_all(x)
            top_subfield[F].contains(x)
        }
    }
}

/// The image of any field homomorphism is contained in the whole-field subfield.
theorem field_hom_image_le_top[F: Field, E: Field](phi: FieldHom[F, E]) {
    subfield_le(field_hom_image(phi), top_subfield[E])
} by {
    top_subfield_ge(field_hom_image(phi))
}

/// The image of a composition of field homomorphisms equals the image of the outer map applied pointwise.
theorem field_hom_image_compose_contains[F: Field, E: Field, K: Field](f: FieldHom[E, K], g: FieldHom[F, E], y: K) {
    field_hom_image(compose_field_hom(f, g)).contains(y) implies field_hom_image(f).contains(y)
} by {
    if field_hom_image(compose_field_hom(f, g)).contains(y) {
        field_hom_image_contains_eq(compose_field_hom(f, g))
        let x: F satisfy { compose_field_hom(f, g).hom(x) = y }
        compose_field_hom_hom(f, g)
        field_hom_image_contains_hom(f, g.hom(x))
        field_hom_image(f).contains(y)
    }
}

/// The image of a composition is contained in the image of the outer map.
theorem field_hom_image_compose_le[F: Field, E: Field, K: Field](f: FieldHom[E, K], g: FieldHom[F, E]) {
    subfield_le(field_hom_image(compose_field_hom(f, g)), field_hom_image(f))
} by {
    forall(y: K) {
        if field_hom_image(compose_field_hom(f, g)).contains(y) {
            field_hom_image_compose_contains(f, g, y)
            field_hom_image(f).contains(y)
        }
    }
}

/// True if phi maps x into the codomain subfield s.
define subfield_preimage_contains[F: Field, E: Field](phi: FieldHom[F, E], s: Subfield[E]) -> F -> Bool {
    function(x: F) { s.contains(phi.hom(x)) }
}

/// The preimage of a codomain subfield under a field homomorphism, as a subfield of the domain.
let subfield_preimage[F: Field, E: Field](phi: FieldHom[F, E], s: Subfield[E]) -> result: Subfield[F] satisfy {
    Subfield.new(subfield_preimage_contains(phi, s)) = Option.some(result)
} by {
    let contains: F -> Bool = subfield_preimage_contains(phi, s)
    forall(x: F) {
        contains(x) = s.contains(phi.hom(x))
    }
    field_hom_zero(phi)
    subfield_contains_zero(s)
    s.contains(phi.hom(F.0))
    contains(F.0)
    subfield_zero_constraint(contains)
    field_hom_one(phi)
    subfield_contains_one(s)
    s.contains(phi.hom(F.1))
    contains(F.1)
    subfield_one_constraint(contains)
    forall(a: F, b: F) {
        if contains(a) and contains(b) {
            s.contains(phi.hom(b))
            subfield_contains_add(s, phi.hom(a), phi.hom(b))
            s.contains(phi.hom(a) + phi.hom(b))
            field_hom_add(phi, a, b)
            contains(a + b)
        }
    }
    subfield_add_constraint(contains)
    forall(a: F) {
        if contains(a) {
            subfield_contains_neg(s, phi.hom(a))
            s.contains(-phi.hom(a))
            field_hom_neg(phi, a)
            contains(-a)
        }
    }
    subfield_neg_constraint(contains)
    subfield_additive_constraint(contains)
    forall(a: F, b: F) {
        if contains(a) and contains(b) {
            s.contains(phi.hom(b))
            subfield_contains_mul(s, phi.hom(a), phi.hom(b))
            s.contains(phi.hom(a) * phi.hom(b))
            field_hom_mul(phi, a, b)
            contains(a * b)
        }
    }
    subfield_mul_constraint(contains)
    forall(a: F) {
        if contains(a) and a != F.0 {
            field_hom_nonzero(phi, a)
            phi.hom(a) != E.0
            subfield_contains_inverse(s, phi.hom(a))
            s.contains(phi.hom(a).inverse)
            field_hom_inverse(phi, a)
            contains(a.inverse)
        }
    }
    subfield_inverse_constraint(contains)
    subfield_multiplicative_constraint(contains)
    subfield_constraint(contains)
}

/// Preimage membership is membership of the image under phi.
theorem subfield_preimage_contains_eq[F: Field, E: Field](phi: FieldHom[F, E], s: Subfield[E], x: F) {
    subfield_preimage(phi, s).contains(x) = s.contains(phi.hom(x))
} by {
    subfield_preimage(phi, s).contains = subfield_preimage_contains(phi, s)
}

/// Preimage of subfields is monotone under inclusion.
theorem subfield_preimage_le[F: Field, E: Field](phi: FieldHom[F, E], s: Subfield[E], t: Subfield[E]) {
    subfield_le(s, t) implies subfield_le(subfield_preimage(phi, s), subfield_preimage(phi, t))
} by {
    if subfield_le(s, t) {
        subfield_le(s, t) = forall(y: E) { s.contains(y) implies t.contains(y) }
        forall(x: F) {
            if subfield_preimage(phi, s).contains(x) {
                subfield_preimage_contains_eq(phi, s, x)
                subfield_preimage_contains_eq(phi, t, x)
                subfield_preimage(phi, t).contains(x)
            }
        }
    }
}

/// Preimage of an intersection of subfields is the intersection of preimages.
theorem subfield_preimage_inter[F: Field, E: Field](phi: FieldHom[F, E], s: Subfield[E], t: Subfield[E]) {
    subfield_preimage(phi, subfield_inter(s, t)) = subfield_inter(subfield_preimage(phi, s), subfield_preimage(phi, t))
} by {
    let lhs = subfield_preimage(phi, subfield_inter(s, t))
    let rhs = subfield_inter(subfield_preimage(phi, s), subfield_preimage(phi, t))
    forall(x: F) {
        subfield_preimage_contains_eq(phi, subfield_inter(s, t), x)
        subfield_inter_contains_eq(s, t, phi.hom(x))
        subfield_preimage_contains_eq(phi, s, x)
        subfield_preimage_contains_eq(phi, t, x)
        subfield_inter_contains_eq(subfield_preimage(phi, s), subfield_preimage(phi, t), x)
        if lhs.contains(x) {
            rhs.contains(x)
        }
        if rhs.contains(x) {
            lhs.contains(x)
        }
        lhs.contains(x) = rhs.contains(x)
    }
    subfield_ext(lhs, rhs)
}

/// Preimage under the identity field homomorphism is the original subfield.
theorem subfield_preimage_identity[F: Field](s: Subfield[F]) {
    subfield_preimage(identity_field_hom[F], s) = s
} by {
    forall(x: F) {
        subfield_preimage_contains_eq(identity_field_hom[F], s, x)
        subfield_preimage(identity_field_hom[F], s).contains(x) = s.contains(x)
    }
    subfield_ext(subfield_preimage(identity_field_hom[F], s), s)
}

/// Preimage under a composition of field homomorphisms is the iterated preimage.
theorem subfield_preimage_compose[F: Field, E: Field, K: Field](f: FieldHom[E, K], g: FieldHom[F, E], s: Subfield[K]) {
    subfield_preimage(compose_field_hom(f, g), s) = subfield_preimage(g, subfield_preimage(f, s))
} by {
    let lhs = subfield_preimage(compose_field_hom(f, g), s)
    let rhs = subfield_preimage(g, subfield_preimage(f, s))
    forall(x: F) {
        subfield_preimage_contains_eq(compose_field_hom(f, g), s, x)
        compose_field_hom_hom(f, g)
        subfield_preimage_contains_eq(g, subfield_preimage(f, s), x)
        subfield_preimage_contains_eq(f, s, g.hom(x))
        lhs.contains(x) = rhs.contains(x)
    }
    subfield_ext(lhs, rhs)
}

/// The preimage of the whole codomain subfield is the whole domain subfield.
theorem subfield_preimage_top[F: Field, E: Field](phi: FieldHom[F, E]) {
    subfield_preimage(phi, top_subfield[E]) = top_subfield[F]
} by {
    forall(x: F) {
        subfield_preimage_contains_eq(phi, top_subfield[E], x)
        top_subfield_contains_all(phi.hom(x))
        top_subfield_contains_all(x)
        subfield_preimage(phi, top_subfield[E]).contains(x) = top_subfield[F].contains(x)
    }
    subfield_ext(subfield_preimage(phi, top_subfield[E]), top_subfield[F])
}

/// True if y is the image under phi of some x in s.
define subfield_image_contains[F: Field, E: Field](phi: FieldHom[F, E], s: Subfield[F]) -> E -> Bool {
    function(y: E) { exists(x: F) { s.contains(x) and phi.hom(x) = y } }
}

/// The forward image of a subfield under a field homomorphism, as a subfield of the codomain.
let subfield_image[F: Field, E: Field](phi: FieldHom[F, E], s: Subfield[F]) -> result: Subfield[E] satisfy {
    Subfield.new(subfield_image_contains(phi, s)) = Option.some(result)
} by {
    let contains: E -> Bool = subfield_image_contains(phi, s)
    forall(y: E) {
        contains(y) = exists(x: F) { s.contains(x) and phi.hom(x) = y }
    }
    subfield_contains_zero(s)
    field_hom_zero(phi)
    s.contains(F.0) and phi.hom(F.0) = E.0
    contains(E.0)
    subfield_zero_constraint(contains)
    subfield_contains_one(s)
    field_hom_one(phi)
    s.contains(F.1) and phi.hom(F.1) = E.1
    contains(E.1)
    subfield_one_constraint(contains)
    forall(a: E, b: E) {
        if contains(a) and contains(b) {
            let xa: F satisfy { s.contains(xa) and phi.hom(xa) = a }
            let xb: F satisfy { s.contains(xb) and phi.hom(xb) = b }
            subfield_contains_add(s, xa, xb)
            field_hom_add(phi, xa, xb)
            phi.hom(xa + xb) = a + b
            contains(a + b)
        }
    }
    subfield_add_constraint(contains)
    forall(a: E) {
        if contains(a) {
            let xa: F satisfy { s.contains(xa) and phi.hom(xa) = a }
            subfield_contains_neg(s, xa)
            field_hom_neg(phi, xa)
            phi.hom(-xa) = -a
            contains(-a)
        }
    }
    subfield_neg_constraint(contains)
    subfield_additive_constraint(contains)
    forall(a: E, b: E) {
        if contains(a) and contains(b) {
            let xa: F satisfy { s.contains(xa) and phi.hom(xa) = a }
            let xb: F satisfy { s.contains(xb) and phi.hom(xb) = b }
            subfield_contains_mul(s, xa, xb)
            field_hom_mul(phi, xa, xb)
            phi.hom(xa * xb) = a * b
            contains(a * b)
        }
    }
    subfield_mul_constraint(contains)
    forall(a: E) {
        if contains(a) and a != E.0 {
            let xa: F satisfy { s.contains(xa) and phi.hom(xa) = a }
            if xa = F.0 {
                field_hom_zero(phi)
                false
            }
            subfield_contains_inverse(s, xa)
            field_hom_inverse(phi, xa)
            phi.hom(xa.inverse) = a.inverse
            contains(a.inverse)
        }
    }
    subfield_inverse_constraint(contains)
    subfield_multiplicative_constraint(contains)
    subfield_constraint(contains)
}

/// Image-subfield membership is exactly the image predicate.
theorem subfield_image_contains_eq[F: Field, E: Field](phi: FieldHom[F, E], s: Subfield[F], y: E) {
    subfield_image(phi, s).contains(y) = exists(x: F) { s.contains(x) and phi.hom(x) = y }
} by {
    subfield_image(phi, s).contains = subfield_image_contains(phi, s)
}

/// The image of an element in s lies in the image subfield.
theorem subfield_image_contains_hom[F: Field, E: Field](phi: FieldHom[F, E], s: Subfield[F], x: F) {
    s.contains(x) implies subfield_image(phi, s).contains(phi.hom(x))
} by {
    if s.contains(x) {
        exists(z: F) { s.contains(z) and phi.hom(z) = phi.hom(x) }
        subfield_image_contains_eq(phi, s, phi.hom(x))
    }
}

/// Forward image of subfields is monotone under inclusion.
theorem subfield_image_le[F: Field, E: Field](phi: FieldHom[F, E], s: Subfield[F], t: Subfield[F]) {
    subfield_le(s, t) implies subfield_le(subfield_image(phi, s), subfield_image(phi, t))
} by {
    if subfield_le(s, t) {
        subfield_le(s, t) = forall(x: F) { s.contains(x) implies t.contains(x) }
        forall(y: E) {
            if subfield_image(phi, s).contains(y) {
                subfield_image_contains_eq(phi, s, y)
                let x: F satisfy { s.contains(x) and phi.hom(x) = y }
                subfield_image_contains_hom(phi, t, x)
                subfield_image(phi, t).contains(phi.hom(x))
                subfield_image(phi, t).contains(y)
            }
        }
    }
}

/// The image of the whole-field subfield is the image of the homomorphism.
theorem subfield_image_top[F: Field, E: Field](phi: FieldHom[F, E]) {
    subfield_image(phi, top_subfield[F]) = field_hom_image(phi)
} by {
    let lhs = subfield_image(phi, top_subfield[F])
    let rhs = field_hom_image(phi)
    forall(y: E) {
        subfield_image_contains_eq(phi, top_subfield[F], y)
        field_hom_image_contains_eq(phi)
        if lhs.contains(y) {
            let x: F satisfy { top_subfield[F].contains(x) and phi.hom(x) = y }
            field_hom_image_contains_hom(phi, x)
            rhs.contains(y)
        }
        if rhs.contains(y) {
            let x: F satisfy { phi.hom(x) = y }
            top_subfield_contains_all(x)
            lhs.contains(y)
        }
        lhs.contains(y) = rhs.contains(y)
    }
    subfield_ext(lhs, rhs)
}

/// Image under the identity field homomorphism is the original subfield.
theorem subfield_image_identity[F: Field](s: Subfield[F]) {
    subfield_image(identity_field_hom[F], s) = s
} by {
    let lhs = subfield_image(identity_field_hom[F], s)
    forall(x: F) {
        subfield_image_contains_eq(identity_field_hom[F], s, x)
        if lhs.contains(x) {
            let z: F satisfy { s.contains(z) and identity_field_hom[F].hom(z) = x }
            identity_field_hom[F].hom(z) = z
            s.contains(x)
        }
        if s.contains(x) {
            exists(z: F) { s.contains(z) and identity_field_hom[F].hom(z) = x }
            lhs.contains(x)
        }
        lhs.contains(x) = s.contains(x)
    }
    subfield_ext(lhs, s)
}

/// Image under a composition is the iterated image.
theorem subfield_image_compose[F: Field, E: Field, K: Field](f: FieldHom[E, K], g: FieldHom[F, E], s: Subfield[F]) {
    subfield_image(compose_field_hom(f, g), s) = subfield_image(f, subfield_image(g, s))
} by {
    let lhs = subfield_image(compose_field_hom(f, g), s)
    let rhs = subfield_image(f, subfield_image(g, s))
    forall(y: K) {
        subfield_image_contains_eq(compose_field_hom(f, g), s, y)
        subfield_image_contains_eq(f, subfield_image(g, s), y)
        if lhs.contains(y) {
            let x: F satisfy { s.contains(x) and compose_field_hom(f, g).hom(x) = y }
            compose_field_hom_hom(f, g)
            subfield_image_contains_hom(g, s, x)
            rhs.contains(y)
        }
        if rhs.contains(y) {
            let z: E satisfy { subfield_image(g, s).contains(z) and f.hom(z) = y }
            subfield_image_contains_eq(g, s, z)
            let x: F satisfy { s.contains(x) and g.hom(x) = z }
            compose_field_hom_hom(f, g)
            f.hom(g.hom(x)) = f.hom(z)
            lhs.contains(y)
        }
        lhs.contains(y) = rhs.contains(y)
    }
    subfield_ext(lhs, rhs)
}

/// The image of an intersection of subfields is contained in the intersection of images (monotonicity direction; no injectivity needed).
theorem subfield_image_inter_le[F: Field, E: Field](phi: FieldHom[F, E], s: Subfield[F], t: Subfield[F]) {
    subfield_le(subfield_image(phi, subfield_inter(s, t)), subfield_inter(subfield_image(phi, s), subfield_image(phi, t)))
} by {
    subfield_inter_le_left(s, t)
    subfield_image_le(phi, subfield_inter(s, t), s)
    subfield_inter_le_right(s, t)
    subfield_image_le(phi, subfield_inter(s, t), t)
    subfield_inter_greatest(subfield_image(phi, s), subfield_image(phi, t), subfield_image(phi, subfield_inter(s, t)))
}

/// Image of an intersection of subfields is the intersection of images (uses injectivity of phi).
theorem subfield_image_inter[F: Field, E: Field](phi: FieldHom[F, E], s: Subfield[F], t: Subfield[F]) {
    subfield_image(phi, subfield_inter(s, t)) = subfield_inter(subfield_image(phi, s), subfield_image(phi, t))
} by {
    let lhs = subfield_image(phi, subfield_inter(s, t))
    let rhs = subfield_inter(subfield_image(phi, s), subfield_image(phi, t))
    forall(y: E) {
        subfield_image_contains_eq(phi, subfield_inter(s, t), y)
        subfield_image_contains_eq(phi, s, y)
        subfield_image_contains_eq(phi, t, y)
        subfield_inter_contains_eq(subfield_image(phi, s), subfield_image(phi, t), y)
        if lhs.contains(y) {
            let x: F satisfy { subfield_inter(s, t).contains(x) and phi.hom(x) = y }
            subfield_inter_contains_eq(s, t, x)
            subfield_image_contains_hom(phi, s, x)
            subfield_image_contains_hom(phi, t, x)
            rhs.contains(y)
        }
        if rhs.contains(y) {
            let xs: F satisfy { s.contains(xs) and phi.hom(xs) = y }
            let xt: F satisfy { t.contains(xt) and phi.hom(xt) = y }
            field_hom_injective(phi, xs, xt)
            xs = xt
            subfield_inter_contains_eq(s, t, xs)
            lhs.contains(y)
        }
        lhs.contains(y) = rhs.contains(y)
    }
    subfield_ext(lhs, rhs)
}

/// The image of any subfield under phi is contained in the image of phi.
theorem subfield_image_le_field_hom_image[F: Field, E: Field](phi: FieldHom[F, E], s: Subfield[F]) {
    subfield_le(subfield_image(phi, s), field_hom_image(phi))
} by {
    forall(y: E) {
        if subfield_image(phi, s).contains(y) {
            subfield_image_contains_eq(phi, s, y)
            let x: F satisfy { s.contains(x) and phi.hom(x) = y }
            field_hom_image_contains_hom(phi, x)
            field_hom_image(phi).contains(y)
        }
    }
}

/// Pointwise membership characterization of the image of a preimage.
theorem subfield_image_preimage_contains[F: Field, E: Field](phi: FieldHom[F, E], t: Subfield[E], y: E) {
    subfield_image(phi, subfield_preimage(phi, t)).contains(y) implies t.contains(y)
} by {
    if subfield_image(phi, subfield_preimage(phi, t)).contains(y) {
        subfield_image_contains_eq(phi, subfield_preimage(phi, t), y)
        let x: F satisfy { subfield_preimage(phi, t).contains(x) and phi.hom(x) = y }
        subfield_preimage_contains_eq(phi, t, x)
        t.contains(y)
    }
}

/// The image of the preimage of t under phi is contained in t.
theorem subfield_image_preimage_le[F: Field, E: Field](phi: FieldHom[F, E], t: Subfield[E]) {
    subfield_le(subfield_image(phi, subfield_preimage(phi, t)), t)
} by {
    forall(y: E) {
        if subfield_image(phi, subfield_preimage(phi, t)).contains(y) {
            subfield_image_preimage_contains(phi, t, y)
            t.contains(y)
        }
    }
    forall(y: E) { subfield_image(phi, subfield_preimage(phi, t)).contains(y) implies t.contains(y) }
    subfield_le_intro(subfield_image(phi, subfield_preimage(phi, t)), t)
}

/// Pointwise membership of an element of s in the preimage of its image.
theorem subfield_preimage_image_contains[F: Field, E: Field](phi: FieldHom[F, E], s: Subfield[F], x: F) {
    s.contains(x) implies subfield_preimage(phi, subfield_image(phi, s)).contains(x)
} by {
    if s.contains(x) {
        subfield_image_contains_hom(phi, s, x)
        subfield_preimage_contains_eq(phi, subfield_image(phi, s), x)
        subfield_preimage(phi, subfield_image(phi, s)).contains(x)
    }
}

/// A subfield is contained in the preimage of its image.
theorem subfield_le_preimage_image[F: Field, E: Field](phi: FieldHom[F, E], s: Subfield[F]) {
    subfield_le(s, subfield_preimage(phi, subfield_image(phi, s)))
} by {
    forall(x: F) {
        if s.contains(x) {
            subfield_preimage_image_contains(phi, s, x)
            subfield_preimage(phi, subfield_image(phi, s)).contains(x)
        }
    }
    forall(x: F) { s.contains(x) implies subfield_preimage(phi, subfield_image(phi, s)).contains(x) }
    subfield_le_intro(s, subfield_preimage(phi, subfield_image(phi, s)))
}

/// The preimage of the image of s under phi is exactly s (uses injectivity of phi).
theorem subfield_preimage_image_eq[F: Field, E: Field](phi: FieldHom[F, E], s: Subfield[F]) {
    subfield_preimage(phi, subfield_image(phi, s)) = s
} by {
    let lhs = subfield_preimage(phi, subfield_image(phi, s))
    forall(x: F) {
        subfield_preimage_contains_eq(phi, subfield_image(phi, s), x)
        subfield_image_contains_eq(phi, s, phi.hom(x))
        if lhs.contains(x) {
            let z: F satisfy { s.contains(z) and phi.hom(z) = phi.hom(x) }
            field_hom_injective(phi, z, x)
            z = x
            s.contains(x)
        }
        if s.contains(x) {
            subfield_image_contains_hom(phi, s, x)
            lhs.contains(x)
        }
        lhs.contains(x) = s.contains(x)
    }
    subfield_ext(lhs, s)
}

/// Pointwise: an element of the intersection of t with the image of phi lies in the image of the preimage.
theorem subfield_inter_preimage_image_contains[F: Field, E: Field](phi: FieldHom[F, E], t: Subfield[E], y: E) {
    t.contains(y) and field_hom_image(phi).contains(y) implies subfield_image(phi, subfield_preimage(phi, t)).contains(y)
} by {
    if t.contains(y) and field_hom_image(phi).contains(y) {
        field_hom_image_contains_eq(phi)
        let x: F satisfy { phi.hom(x) = y }
        subfield_preimage_contains_eq(phi, t, x)
        subfield_image_contains_eq(phi, subfield_preimage(phi, t), y)
        subfield_image(phi, subfield_preimage(phi, t)).contains(y)
    }
}

/// Pointwise: an element of the image of the preimage lies in the image of phi.
theorem subfield_image_preimage_in_image[F: Field, E: Field](phi: FieldHom[F, E], t: Subfield[E], y: E) {
    subfield_image(phi, subfield_preimage(phi, t)).contains(y) implies field_hom_image(phi).contains(y)
} by {
    if subfield_image(phi, subfield_preimage(phi, t)).contains(y) {
        subfield_image_contains_eq(phi, subfield_preimage(phi, t), y)
        let x: F satisfy { subfield_preimage(phi, t).contains(x) and phi.hom(x) = y }
        field_hom_image_contains_hom(phi, x)
        field_hom_image(phi).contains(y)
    }
}

/// The image of the preimage of t under phi equals the intersection of t with the image of phi.
theorem subfield_image_preimage_eq_inter[F: Field, E: Field](phi: FieldHom[F, E], t: Subfield[E]) {
    subfield_image(phi, subfield_preimage(phi, t)) = subfield_inter(t, field_hom_image(phi))
} by {
    let lhs = subfield_image(phi, subfield_preimage(phi, t))
    let rhs = subfield_inter(t, field_hom_image(phi))
    forall(y: E) {
        subfield_inter_contains_eq(t, field_hom_image(phi), y)
        if lhs.contains(y) {
            subfield_image_preimage_contains(phi, t, y)
            subfield_image_preimage_in_image(phi, t, y)
            field_hom_image(phi).contains(y)
            rhs.contains(y)
        }
        if rhs.contains(y) {
            field_hom_image(phi).contains(y)
            subfield_inter_preimage_image_contains(phi, t, y)
            lhs.contains(y)
        }
        lhs.contains(y) = rhs.contains(y)
    }
    subfield_ext(lhs, rhs)
    lhs = rhs
}

/// Galois connection: image-preimage adjunction (forward direction).
theorem subfield_image_le_of_le_preimage[F: Field, E: Field](phi: FieldHom[F, E], s: Subfield[F], t: Subfield[E]) {
    subfield_le(s, subfield_preimage(phi, t)) implies subfield_le(subfield_image(phi, s), t)
} by {
    if subfield_le(s, subfield_preimage(phi, t)) {
        subfield_image_le(phi, s, subfield_preimage(phi, t))
        subfield_image_preimage_le(phi, t)
        subfield_le(subfield_image(phi, subfield_preimage(phi, t)), t)
        subfield_le_trans(subfield_image(phi, s), subfield_image(phi, subfield_preimage(phi, t)), t)
    }
}

/// Galois connection: image-preimage adjunction (backward direction).
theorem subfield_le_preimage_of_image_le[F: Field, E: Field](phi: FieldHom[F, E], s: Subfield[F], t: Subfield[E]) {
    subfield_le(subfield_image(phi, s), t) implies subfield_le(s, subfield_preimage(phi, t))
} by {
    if subfield_le(subfield_image(phi, s), t) {
        subfield_le_preimage_image(phi, s)
        subfield_preimage_le(phi, subfield_image(phi, s), t)
        subfield_le(subfield_preimage(phi, subfield_image(phi, s)), subfield_preimage(phi, t))
        subfield_le_trans(s, subfield_preimage(phi, subfield_image(phi, s)), subfield_preimage(phi, t))
    }
}

/// Galois connection between subfield_image and subfield_preimage.
theorem subfield_image_le_iff_le_preimage[F: Field, E: Field](phi: FieldHom[F, E], s: Subfield[F], t: Subfield[E]) {
    subfield_le(subfield_image(phi, s), t) = subfield_le(s, subfield_preimage(phi, t))
} by {
    if subfield_le(subfield_image(phi, s), t) {
        subfield_le_preimage_of_image_le(phi, s, t)
        subfield_le(s, subfield_preimage(phi, t))
    }
    if subfield_le(s, subfield_preimage(phi, t)) {
        subfield_image_le_of_le_preimage(phi, s, t)
        subfield_le(subfield_image(phi, s), t)
    }
}

/// A codomain subfield is the image of its preimage exactly when it lies in the image of phi.
theorem subfield_image_preimage_eq_iff_le_image[F: Field, E: Field](phi: FieldHom[F, E], t: Subfield[E]) {
    (subfield_image(phi, subfield_preimage(phi, t)) = t) = subfield_le(t, field_hom_image(phi))
} by {
    subfield_image_preimage_eq_inter(phi, t)
    subfield_inter_eq_left_iff_le(t, field_hom_image(phi))
}

/// The image-preimage-image closure operator is idempotent on subfields of the domain.
theorem subfield_image_preimage_image_eq[F: Field, E: Field](phi: FieldHom[F, E], s: Subfield[F]) {
    subfield_image(phi, subfield_preimage(phi, subfield_image(phi, s))) = subfield_image(phi, s)
} by {
    subfield_image_preimage_le(phi, subfield_image(phi, s))
    subfield_le_preimage_image(phi, s)
    subfield_image_le(phi, s, subfield_preimage(phi, subfield_image(phi, s)))
    subfield_le_antisymm(subfield_image(phi, subfield_preimage(phi, subfield_image(phi, s))), subfield_image(phi, s))
}

/// The preimage-image-preimage closure operator is idempotent on subfields of the codomain.
theorem subfield_preimage_image_preimage_eq[F: Field, E: Field](phi: FieldHom[F, E], t: Subfield[E]) {
    subfield_preimage(phi, subfield_image(phi, subfield_preimage(phi, t))) = subfield_preimage(phi, t)
} by {
    subfield_le_preimage_image(phi, subfield_preimage(phi, t))
    subfield_image_preimage_le(phi, t)
    subfield_preimage_le(phi, subfield_image(phi, subfield_preimage(phi, t)), t)
    subfield_le_antisymm(subfield_preimage(phi, subfield_image(phi, subfield_preimage(phi, t))), subfield_preimage(phi, t))
}

/// The image of a composition equals the outer map's image of the inner map's image.
theorem field_hom_image_compose_eq[F: Field, E: Field, K: Field](f: FieldHom[E, K], g: FieldHom[F, E]) {
    field_hom_image(compose_field_hom(f, g)) = subfield_image(f, field_hom_image(g))
} by {
    let lhs = field_hom_image(compose_field_hom(f, g))
    let rhs = subfield_image(f, field_hom_image(g))
    forall(y: K) {
        if lhs.contains(y) {
            field_hom_image_contains_eq(compose_field_hom(f, g))
            let x: F satisfy { compose_field_hom(f, g).hom(x) = y }
            compose_field_hom_hom(f, g)
            field_hom_image_contains_hom(g, x)
            subfield_image_contains_hom(f, field_hom_image(g), g.hom(x))
            rhs.contains(y)
        }
        if rhs.contains(y) {
            subfield_image_contains_eq(f, field_hom_image(g), y)
            let z: E satisfy { field_hom_image(g).contains(z) and f.hom(z) = y }
            field_hom_image_contains_eq(g)
            let x: F satisfy { g.hom(x) = z }
            compose_field_hom_hom(f, g)
            f.hom(g.hom(x)) = f.hom(z)
            field_hom_image_contains_hom(compose_field_hom(f, g), x)
            lhs.contains(y)
        }
        lhs.contains(y) = rhs.contains(y)
    }
    subfield_ext(lhs, rhs)
}

/// True if every element of the codomain is hit by phi.
define is_field_hom_surjective[F: Field, E: Field](phi: FieldHom[F, E]) -> Bool {
    forall(y: E) { exists(x: F) { phi.hom(x) = y } }
}

/// If phi's image equals the whole codomain then phi is surjective.
theorem field_hom_image_eq_top_implies_surjective[F: Field, E: Field](phi: FieldHom[F, E]) {
    field_hom_image(phi) = top_subfield[E] implies is_field_hom_surjective(phi)
} by {
    if field_hom_image(phi) = top_subfield[E] {
        field_hom_image_contains_eq(phi)
        forall(y: E) {
            top_subfield_contains_all(y)
            exists(x: F) { phi.hom(x) = y }
        }
        forall(y: E) { exists(x: F) { phi.hom(x) = y } }
        is_field_hom_surjective(phi)
    }
}

/// If phi is surjective then its image equals the whole codomain.
theorem surjective_implies_field_hom_image_eq_top[F: Field, E: Field](phi: FieldHom[F, E]) {
    is_field_hom_surjective(phi) implies field_hom_image(phi) = top_subfield[E]
} by {
    if is_field_hom_surjective(phi) {
        forall(y: E) {
            let x: F satisfy { phi.hom(x) = y }
            field_hom_image_contains_hom(phi, x)
            top_subfield_contains_all(y)
            field_hom_image(phi).contains(y) = top_subfield[E].contains(y)
        }
        subfield_ext(field_hom_image(phi), top_subfield[E])
        field_hom_image(phi) = top_subfield[E]
    }
}

/// The image of phi is the whole codomain exactly when phi is surjective.
theorem field_hom_image_eq_top_iff_surjective[F: Field, E: Field](phi: FieldHom[F, E]) {
    (field_hom_image(phi) = top_subfield[E]) = is_field_hom_surjective(phi)
} by {
    field_hom_image_eq_top_implies_surjective(phi)
    surjective_implies_field_hom_image_eq_top(phi)
}

/// The identity field homomorphism is surjective.
theorem identity_field_hom_surjective[F: Field] {
    is_field_hom_surjective(identity_field_hom[F])
} by {
    field_hom_image_identity[F]
    field_hom_image_eq_top_iff_surjective(identity_field_hom[F])
}

/// The composition of two surjective field homomorphisms is surjective.
theorem compose_field_hom_surjective[F: Field, E: Field, K: Field](f: FieldHom[E, K], g: FieldHom[F, E]) {
    is_field_hom_surjective(f) and is_field_hom_surjective(g) implies is_field_hom_surjective(compose_field_hom(f, g))
} by {
    if is_field_hom_surjective(f) and is_field_hom_surjective(g) {
        forall(z: K) {
            let y: E satisfy { f.hom(y) = z }
            let x: F satisfy { g.hom(x) = y }
            compose_field_hom_hom(f, g)
            f.hom(g.hom(x)) = f.hom(y)
            compose_field_hom(f, g).hom(x) = z
            exists(w: F) { compose_field_hom(f, g).hom(w) = z }
        }
        forall(z: K) { exists(w: F) { compose_field_hom(f, g).hom(w) = z } }
        is_field_hom_surjective(compose_field_hom(f, g))
    }
}
