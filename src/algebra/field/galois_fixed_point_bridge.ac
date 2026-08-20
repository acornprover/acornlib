from order import PartialOrder
from order import is_monotone, monotone_step
from algebra.field.galois_connection import is_galois_connection, galois_closure, galois_kernel,
    galois_closure_at, galois_kernel_at,
    galois_connection_le_upper,
    galois_connection_lower_monotone, galois_connection_upper_monotone,
    is_galois_insertion, is_galois_coinsertion,
    galois_insertion_kernel_eq_self, galois_coinsertion_closure_eq_self
from analysis import is_fixed_point

/// Being fixed by the closure induced from `lower ⊣ upper` unfolds to the
/// inverse identity `upper(lower(a)) = a`.
theorem galois_closure_fixed_upper_lower_eq[A: PartialOrder, B: PartialOrder](
    lower: A -> B,
    upper: B -> A,
    a: A
) {
    is_fixed_point(galois_closure(lower, upper), a) implies upper(lower(a)) = a
} by {
    if is_fixed_point(galois_closure(lower, upper), a) {
        galois_closure_at(lower, upper, a)
        upper(lower(a)) = a
    }
}

/// Being fixed by the kernel induced from `lower ⊣ upper` unfolds to the
/// inverse identity `lower(upper(b)) = b`.
theorem galois_kernel_fixed_lower_upper_eq[A: PartialOrder, B: PartialOrder](
    lower: A -> B,
    upper: B -> A,
    b: B
) {
    is_fixed_point(galois_kernel(lower, upper), b) implies lower(upper(b)) = b
} by {
    if is_fixed_point(galois_kernel(lower, upper), b) {
        galois_kernel_at(lower, upper, b)
        lower(upper(b)) = b
    }
}

/// The lower adjoint sends closure-fixed points to kernel-fixed points.
theorem galois_lower_maps_closure_fixed_to_kernel_fixed[A: PartialOrder, B: PartialOrder](
    lower: A -> B,
    upper: B -> A,
    a: A
) {
    is_galois_connection(lower, upper) and is_fixed_point(galois_closure(lower, upper), a)
        implies is_fixed_point(galois_kernel(lower, upper), lower(a))
} by {
    if is_galois_connection(lower, upper) and is_fixed_point(galois_closure(lower, upper), a) {
        galois_closure_fixed_upper_lower_eq(lower, upper, a)
        galois_kernel_at(lower, upper, lower(a))
        is_fixed_point(galois_kernel(lower, upper), lower(a))
    }
}

/// The upper adjoint sends kernel-fixed points to closure-fixed points.
theorem galois_upper_maps_kernel_fixed_to_closure_fixed[A: PartialOrder, B: PartialOrder](
    lower: A -> B,
    upper: B -> A,
    b: B
) {
    is_galois_connection(lower, upper) and is_fixed_point(galois_kernel(lower, upper), b)
        implies is_fixed_point(galois_closure(lower, upper), upper(b))
} by {
    if is_galois_connection(lower, upper) and is_fixed_point(galois_kernel(lower, upper), b) {
        galois_kernel_fixed_lower_upper_eq(lower, upper, b)
        galois_closure_at(lower, upper, upper(b))
        is_fixed_point(galois_closure(lower, upper), upper(b))
    }
}

/// On closure-fixed points, `upper` is a left inverse to `lower`.
theorem galois_fixed_lower_then_upper[A: PartialOrder, B: PartialOrder](
    lower: A -> B,
    upper: B -> A,
    a: A
) {
    is_galois_connection(lower, upper) and is_fixed_point(galois_closure(lower, upper), a)
        implies upper(lower(a)) = a
} by {
    if is_galois_connection(lower, upper) and is_fixed_point(galois_closure(lower, upper), a) {
        galois_closure_fixed_upper_lower_eq(lower, upper, a)
    }
}

/// On kernel-fixed points, `lower` is a left inverse to `upper`.
theorem galois_fixed_upper_then_lower[A: PartialOrder, B: PartialOrder](
    lower: A -> B,
    upper: B -> A,
    b: B
) {
    is_galois_connection(lower, upper) and is_fixed_point(galois_kernel(lower, upper), b)
        implies lower(upper(b)) = b
} by {
    if is_galois_connection(lower, upper) and is_fixed_point(galois_kernel(lower, upper), b) {
        galois_kernel_fixed_lower_upper_eq(lower, upper, b)
    }
}

/// The lower adjoint preserves order between closure-fixed points.
theorem galois_lower_preserves_order_on_closure_fixed[A: PartialOrder, B: PartialOrder](
    lower: A -> B,
    upper: B -> A,
    a1: A,
    a2: A
) {
    is_galois_connection(lower, upper) and
    is_fixed_point(galois_closure(lower, upper), a1) and
    is_fixed_point(galois_closure(lower, upper), a2) and
    a1 <= a2 implies lower(a1) <= lower(a2)
} by {
    if is_galois_connection(lower, upper) and
        is_fixed_point(galois_closure(lower, upper), a1) and
        is_fixed_point(galois_closure(lower, upper), a2) and
        a1 <= a2 {
        galois_connection_lower_monotone(lower, upper)
        monotone_step(lower, a1, a2)
        lower(a1) <= lower(a2)
    }
}

/// The lower adjoint reflects order between closure-fixed points.
theorem galois_lower_reflects_order_on_closure_fixed[A: PartialOrder, B: PartialOrder](
    lower: A -> B,
    upper: B -> A,
    a1: A,
    a2: A
) {
    is_galois_connection(lower, upper) and
    is_fixed_point(galois_closure(lower, upper), a1) and
    is_fixed_point(galois_closure(lower, upper), a2) and
    lower(a1) <= lower(a2) implies a1 <= a2
} by {
    if is_galois_connection(lower, upper) and
        is_fixed_point(galois_closure(lower, upper), a1) and
        is_fixed_point(galois_closure(lower, upper), a2) and
        lower(a1) <= lower(a2) {
        galois_connection_le_upper(lower, upper, a1, lower(a2))
        galois_fixed_lower_then_upper(lower, upper, a2)
        upper(lower(a2)) = a2
        a1 <= a2
    }
}

/// On closure-fixed points, the lower adjoint is order-reflecting and order-preserving.
theorem galois_lower_order_iff_on_closure_fixed[A: PartialOrder, B: PartialOrder](
    lower: A -> B,
    upper: B -> A,
    a1: A,
    a2: A
) {
    is_galois_connection(lower, upper) and
    is_fixed_point(galois_closure(lower, upper), a1) and
    is_fixed_point(galois_closure(lower, upper), a2) implies
    (lower(a1) <= lower(a2) = (a1 <= a2))
} by {
    if is_galois_connection(lower, upper) and
        is_fixed_point(galois_closure(lower, upper), a1) and
        is_fixed_point(galois_closure(lower, upper), a2) {
        if lower(a1) <= lower(a2) {
            galois_lower_reflects_order_on_closure_fixed(lower, upper, a1, a2)
            a1 <= a2
        }
        if a1 <= a2 {
            galois_lower_preserves_order_on_closure_fixed(lower, upper, a1, a2)
            lower(a1) <= lower(a2)
        }
        lower(a1) <= lower(a2) = (a1 <= a2)
    }
}

/// The upper adjoint preserves order between kernel-fixed points.
theorem galois_upper_preserves_order_on_kernel_fixed[A: PartialOrder, B: PartialOrder](
    lower: A -> B,
    upper: B -> A,
    b1: B,
    b2: B
) {
    is_galois_connection(lower, upper) and
    is_fixed_point(galois_kernel(lower, upper), b1) and
    is_fixed_point(galois_kernel(lower, upper), b2) and
    b1 <= b2 implies upper(b1) <= upper(b2)
} by {
    if is_galois_connection(lower, upper) and
        is_fixed_point(galois_kernel(lower, upper), b1) and
        is_fixed_point(galois_kernel(lower, upper), b2) and
        b1 <= b2 {
        galois_connection_upper_monotone(lower, upper)
        monotone_step(upper, b1, b2)
        upper(b1) <= upper(b2)
    }
}

/// The upper adjoint reflects order between kernel-fixed points.
theorem galois_upper_reflects_order_on_kernel_fixed[A: PartialOrder, B: PartialOrder](
    lower: A -> B,
    upper: B -> A,
    b1: B,
    b2: B
) {
    is_galois_connection(lower, upper) and
    is_fixed_point(galois_kernel(lower, upper), b1) and
    is_fixed_point(galois_kernel(lower, upper), b2) and
    upper(b1) <= upper(b2) implies b1 <= b2
} by {
    if is_galois_connection(lower, upper) and
        is_fixed_point(galois_kernel(lower, upper), b1) and
        is_fixed_point(galois_kernel(lower, upper), b2) and
        upper(b1) <= upper(b2) {
        galois_connection_lower_monotone(lower, upper)
        monotone_step(lower, upper(b1), upper(b2))
        lower(upper(b1)) <= lower(upper(b2))
        galois_fixed_upper_then_lower(lower, upper, b1)
        lower(upper(b1)) = b1
        galois_fixed_upper_then_lower(lower, upper, b2)
        lower(upper(b2)) = b2
        b1 <= b2
    }
}

/// On kernel-fixed points, the upper adjoint is order-reflecting and order-preserving.
theorem galois_upper_order_iff_on_kernel_fixed[A: PartialOrder, B: PartialOrder](
    lower: A -> B,
    upper: B -> A,
    b1: B,
    b2: B
) {
    is_galois_connection(lower, upper) and
    is_fixed_point(galois_kernel(lower, upper), b1) and
    is_fixed_point(galois_kernel(lower, upper), b2) implies
    (upper(b1) <= upper(b2) = (b1 <= b2))
} by {
    if is_galois_connection(lower, upper) and
        is_fixed_point(galois_kernel(lower, upper), b1) and
        is_fixed_point(galois_kernel(lower, upper), b2) {
        if upper(b1) <= upper(b2) {
            galois_upper_reflects_order_on_kernel_fixed(lower, upper, b1, b2)
            b1 <= b2
        }
        if b1 <= b2 {
            galois_upper_preserves_order_on_kernel_fixed(lower, upper, b1, b2)
            upper(b1) <= upper(b2)
        }
        upper(b1) <= upper(b2) = (b1 <= b2)
    }
}

/// For selected closure/kernel fixed points, the lower-image equality is
/// equivalent to the upper-preimage equality. This is the concrete inverse-pair
/// bridge behind the fixed-point correspondence.
theorem galois_fixed_lower_eq_iff_upper_eq[A: PartialOrder, B: PartialOrder](
    lower: A -> B,
    upper: B -> A,
    a: A,
    b: B
) {
    is_galois_connection(lower, upper) and
    is_fixed_point(galois_closure(lower, upper), a) and
    is_fixed_point(galois_kernel(lower, upper), b) implies
    (lower(a) = b = (a = upper(b)))
} by {
    if is_galois_connection(lower, upper) and
        is_fixed_point(galois_closure(lower, upper), a) and
        is_fixed_point(galois_kernel(lower, upper), b) {
        galois_fixed_lower_then_upper(lower, upper, a)
        galois_fixed_upper_then_lower(lower, upper, b)
        if lower(a) = b {
            a = upper(b)
        }
        if a = upper(b) {
            lower(a) = b
        }
        lower(a) = b = (a = upper(b))
    }
}

/// In a Galois insertion, every upper-side point is kernel-fixed.
theorem galois_insertion_every_kernel_fixed[A: PartialOrder, B: PartialOrder](
    lower: A -> B,
    upper: B -> A,
    b: B
) {
    is_galois_insertion(lower, upper) implies is_fixed_point(galois_kernel(lower, upper), b)
} by {
    if is_galois_insertion(lower, upper) {
        galois_insertion_kernel_eq_self(lower, upper, b)
        is_fixed_point(galois_kernel(lower, upper), b)
    }
}

/// In a Galois coinsertion, every lower-side point is closure-fixed.
theorem galois_coinsertion_every_closure_fixed[A: PartialOrder, B: PartialOrder](
    lower: A -> B,
    upper: B -> A,
    a: A
) {
    is_galois_coinsertion(lower, upper) implies is_fixed_point(galois_closure(lower, upper), a)
} by {
    if is_galois_coinsertion(lower, upper) {
        galois_coinsertion_closure_eq_self(lower, upper, a)
        is_fixed_point(galois_closure(lower, upper), a)
    }
}
