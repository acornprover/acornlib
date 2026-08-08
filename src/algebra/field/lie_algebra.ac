from comm_ring import CommRing
from algebra.add_comm_group import AddCommGroup
from algebra.add_group import left_cancel, inverse_inverse
from algebra.module.module import Module, module_smul_zero_left, module_smul_zero_right
from algebra.module.submodule import Submodule,
    submodule_contains_zero, submodule_contains_add, submodule_contains_neg,
    submodule_contains_smul

/// True if a bracket is left-additive: [x + y, z] = [x, z] + [y, z].
define lie_bracket_add_left_constraint[R: CommRing, L: AddCommGroup](
    bracket: (L, L) -> L
) -> Bool {
    forall(x: L, y: L, z: L) {
        bracket(x + y, z) = bracket(x, z) + bracket(y, z)
    }
}

/// True if a bracket is right-additive: [x, y + z] = [x, y] + [x, z].
define lie_bracket_add_right_constraint[R: CommRing, L: AddCommGroup](
    bracket: (L, L) -> L
) -> Bool {
    forall(x: L, y: L, z: L) {
        bracket(x, y + z) = bracket(x, y) + bracket(x, z)
    }
}

/// True if a bracket is left-scalar-linear: [c.x, y] = c.[x, y].
define lie_bracket_smul_left_constraint[R: CommRing, L: AddCommGroup](
    m: Module[R, L], bracket: (L, L) -> L
) -> Bool {
    forall(c: R, x: L, y: L) {
        bracket(m.smul(c, x), y) = m.smul(c, bracket(x, y))
    }
}

/// True if a bracket is right-scalar-linear: [x, c.y] = c.[x, y].
define lie_bracket_smul_right_constraint[R: CommRing, L: AddCommGroup](
    m: Module[R, L], bracket: (L, L) -> L
) -> Bool {
    forall(c: R, x: L, y: L) {
        bracket(x, m.smul(c, y)) = m.smul(c, bracket(x, y))
    }
}

/// True if a bracket is alternating: [x, x] = 0.
define lie_bracket_alternating_constraint[R: CommRing, L: AddCommGroup](
    bracket: (L, L) -> L
) -> Bool {
    forall(x: L) {
        bracket(x, x) = L.0
    }
}

/// True if a bracket satisfies the Jacobi identity.
define lie_bracket_jacobi_constraint[R: CommRing, L: AddCommGroup](
    bracket: (L, L) -> L
) -> Bool {
    forall(x: L, y: L, z: L) {
        (bracket(x, bracket(y, z)) + bracket(y, bracket(z, x))) + bracket(z, bracket(x, y)) = L.0
    }
}

/// True if a bracket together with a module structure forms a Lie algebra.
define is_lie_bracket[R: CommRing, L: AddCommGroup](
    m: Module[R, L], bracket: (L, L) -> L
) -> Bool {
    lie_bracket_add_left_constraint[R, L](bracket)
    and lie_bracket_add_right_constraint[R, L](bracket)
    and lie_bracket_smul_left_constraint(m, bracket)
    and lie_bracket_smul_right_constraint(m, bracket)
    and lie_bracket_alternating_constraint[R, L](bracket)
    and lie_bracket_jacobi_constraint[R, L](bracket)
}

/// A Lie algebra over R: an R-module L equipped with a bilinear bracket that is
/// alternating and satisfies the Jacobi identity.
structure LieAlgebra[R: CommRing, L: AddCommGroup] {
    /// The underlying R-module structure.
    module: Module[R, L]
    /// The Lie bracket.
    bracket: (L, L) -> L
} constraint {
    is_lie_bracket(module, bracket)
}

/// The bracket is left-additive.
theorem lie_bracket_add_left[R: CommRing, L: AddCommGroup](
    a: LieAlgebra[R, L], x: L, y: L, z: L
) {
    a.bracket(x + y, z) = a.bracket(x, z) + a.bracket(y, z)
} by {
    is_lie_bracket(a.module, a.bracket) = (lie_bracket_add_left_constraint[R, L](a.bracket)
        and lie_bracket_add_right_constraint[R, L](a.bracket)
        and lie_bracket_smul_left_constraint(a.module, a.bracket)
        and lie_bracket_smul_right_constraint(a.module, a.bracket)
        and lie_bracket_alternating_constraint[R, L](a.bracket)
        and lie_bracket_jacobi_constraint[R, L](a.bracket))
    lie_bracket_add_left_constraint[R, L](a.bracket) = forall(u: L, v: L, w: L) {
        a.bracket(u + v, w) = a.bracket(u, w) + a.bracket(v, w)
    }
}

/// The bracket is right-additive.
theorem lie_bracket_add_right[R: CommRing, L: AddCommGroup](
    a: LieAlgebra[R, L], x: L, y: L, z: L
) {
    a.bracket(x, y + z) = a.bracket(x, y) + a.bracket(x, z)
} by {
    is_lie_bracket(a.module, a.bracket) = (lie_bracket_add_left_constraint[R, L](a.bracket)
        and lie_bracket_add_right_constraint[R, L](a.bracket)
        and lie_bracket_smul_left_constraint(a.module, a.bracket)
        and lie_bracket_smul_right_constraint(a.module, a.bracket)
        and lie_bracket_alternating_constraint[R, L](a.bracket)
        and lie_bracket_jacobi_constraint[R, L](a.bracket))
    lie_bracket_add_right_constraint[R, L](a.bracket) = forall(u: L, v: L, w: L) {
        a.bracket(u, v + w) = a.bracket(u, v) + a.bracket(u, w)
    }
}

/// The bracket is left-linear in the scalar.
theorem lie_bracket_smul_left[R: CommRing, L: AddCommGroup](
    a: LieAlgebra[R, L], c: R, x: L, y: L
) {
    a.bracket(a.module.smul(c, x), y) = a.module.smul(c, a.bracket(x, y))
} by {
    is_lie_bracket(a.module, a.bracket) = (lie_bracket_add_left_constraint[R, L](a.bracket)
        and lie_bracket_add_right_constraint[R, L](a.bracket)
        and lie_bracket_smul_left_constraint(a.module, a.bracket)
        and lie_bracket_smul_right_constraint(a.module, a.bracket)
        and lie_bracket_alternating_constraint[R, L](a.bracket)
        and lie_bracket_jacobi_constraint[R, L](a.bracket))
    lie_bracket_smul_left_constraint(a.module, a.bracket) = forall(d: R, u: L, v: L) {
        a.bracket(a.module.smul(d, u), v) = a.module.smul(d, a.bracket(u, v))
    }
}

/// The bracket is right-linear in the scalar.
theorem lie_bracket_smul_right[R: CommRing, L: AddCommGroup](
    a: LieAlgebra[R, L], c: R, x: L, y: L
) {
    a.bracket(x, a.module.smul(c, y)) = a.module.smul(c, a.bracket(x, y))
} by {
    is_lie_bracket(a.module, a.bracket) = (lie_bracket_add_left_constraint[R, L](a.bracket)
        and lie_bracket_add_right_constraint[R, L](a.bracket)
        and lie_bracket_smul_left_constraint(a.module, a.bracket)
        and lie_bracket_smul_right_constraint(a.module, a.bracket)
        and lie_bracket_alternating_constraint[R, L](a.bracket)
        and lie_bracket_jacobi_constraint[R, L](a.bracket))
    lie_bracket_smul_right_constraint(a.module, a.bracket) = forall(d: R, u: L, v: L) {
        a.bracket(u, a.module.smul(d, v)) = a.module.smul(d, a.bracket(u, v))
    }
}

/// The bracket vanishes on equal arguments.
theorem lie_bracket_self[R: CommRing, L: AddCommGroup](a: LieAlgebra[R, L], x: L) {
    a.bracket(x, x) = L.0
} by {
    is_lie_bracket(a.module, a.bracket) = (lie_bracket_add_left_constraint[R, L](a.bracket)
        and lie_bracket_add_right_constraint[R, L](a.bracket)
        and lie_bracket_smul_left_constraint(a.module, a.bracket)
        and lie_bracket_smul_right_constraint(a.module, a.bracket)
        and lie_bracket_alternating_constraint[R, L](a.bracket)
        and lie_bracket_jacobi_constraint[R, L](a.bracket))
    lie_bracket_alternating_constraint[R, L](a.bracket) = forall(u: L) {
        a.bracket(u, u) = L.0
    }
}

/// The bracket satisfies the Jacobi identity.
theorem lie_bracket_jacobi[R: CommRing, L: AddCommGroup](
    a: LieAlgebra[R, L], x: L, y: L, z: L
) {
    (a.bracket(x, a.bracket(y, z)) + a.bracket(y, a.bracket(z, x))) + a.bracket(z, a.bracket(x, y)) = L.0
} by {
    is_lie_bracket(a.module, a.bracket) = (lie_bracket_add_left_constraint[R, L](a.bracket)
        and lie_bracket_add_right_constraint[R, L](a.bracket)
        and lie_bracket_smul_left_constraint(a.module, a.bracket)
        and lie_bracket_smul_right_constraint(a.module, a.bracket)
        and lie_bracket_alternating_constraint[R, L](a.bracket)
        and lie_bracket_jacobi_constraint[R, L](a.bracket))
    lie_bracket_jacobi_constraint[R, L](a.bracket) = forall(u: L, v: L, w: L) {
        (a.bracket(u, a.bracket(v, w)) + a.bracket(v, a.bracket(w, u))) + a.bracket(w, a.bracket(u, v)) = L.0
    }
}

/// The bracket sends a zero left argument to zero.
theorem lie_bracket_zero_left[R: CommRing, L: AddCommGroup](a: LieAlgebra[R, L], x: L) {
    a.bracket(L.0, x) = L.0
} by {
    lie_bracket_add_left(a, L.0, L.0, x)
    left_cancel(a.bracket(L.0, x), a.bracket(L.0, x), L.0)
}

/// The bracket sends a zero right argument to zero.
theorem lie_bracket_zero_right[R: CommRing, L: AddCommGroup](a: LieAlgebra[R, L], x: L) {
    a.bracket(x, L.0) = L.0
} by {
    lie_bracket_add_right(a, x, L.0, L.0)
    left_cancel(a.bracket(x, L.0), a.bracket(x, L.0), L.0)
}

/// Skew-symmetry: [x, y] = -[y, x].
theorem lie_bracket_skew[R: CommRing, L: AddCommGroup](a: LieAlgebra[R, L], x: L, y: L) {
    a.bracket(x, y) = -a.bracket(y, x)
} by {
    lie_bracket_self(a, x + y)
    lie_bracket_add_left(a, x, y, x + y)
    lie_bracket_add_right(a, x, x, y)
    lie_bracket_add_right(a, y, x, y)
    lie_bracket_self(a, x)
    lie_bracket_self(a, y)
    a.bracket(x, x + y) = a.bracket(x, x) + a.bracket(x, y)
    a.bracket(x, x + y) = L.0 + a.bracket(x, y)
    a.bracket(x, x + y) = a.bracket(x, y)
    a.bracket(y, x + y) = a.bracket(y, x) + a.bracket(y, y)
    a.bracket(y, x + y) = a.bracket(y, x) + L.0
    a.bracket(y, x + y) = a.bracket(y, x)
    a.bracket(x + y, x + y) = a.bracket(x, x + y) + a.bracket(y, x + y)
    L.0 = a.bracket(x, y) + a.bracket(y, x)
    a.bracket(x, y) + a.bracket(y, x) = a.bracket(y, x) + -a.bracket(y, x)
}

/// The bracket negates in the left argument.
theorem lie_bracket_neg_left[R: CommRing, L: AddCommGroup](a: LieAlgebra[R, L], x: L, y: L) {
    a.bracket(-x, y) = -a.bracket(x, y)
} by {
    lie_bracket_add_left(a, x, -x, y)
    lie_bracket_zero_left(a, y)
    left_cancel(a.bracket(x, y), a.bracket(-x, y), -a.bracket(x, y))
}

/// The bracket negates in the right argument.
theorem lie_bracket_neg_right[R: CommRing, L: AddCommGroup](a: LieAlgebra[R, L], x: L, y: L) {
    a.bracket(x, -y) = -a.bracket(x, y)
} by {
    lie_bracket_add_right(a, x, y, -y)
    lie_bracket_zero_right(a, x)
    left_cancel(a.bracket(x, y), a.bracket(x, -y), -a.bracket(x, y))
}

/// The bracket is left-additive, as a rewrite-oriented endpoint.
theorem lie_bracket_add_left_eq[R: CommRing, L: AddCommGroup](
    a: LieAlgebra[R, L], x: L, y: L, z: L
) {
    a.bracket(x + y, z) = a.bracket(x, z) + a.bracket(y, z)
} by {
    lie_bracket_add_left(a, x, y, z)
}

/// The bracket is right-additive, as a rewrite-oriented endpoint.
theorem lie_bracket_add_right_eq[R: CommRing, L: AddCommGroup](
    a: LieAlgebra[R, L], x: L, y: L, z: L
) {
    a.bracket(x, y + z) = a.bracket(x, y) + a.bracket(x, z)
} by {
    lie_bracket_add_right(a, x, y, z)
}

/// The bracket is left-scalar-linear, as a rewrite-oriented endpoint.
theorem lie_bracket_smul_left_eq[R: CommRing, L: AddCommGroup](
    a: LieAlgebra[R, L], c: R, x: L, y: L
) {
    a.bracket(a.module.smul(c, x), y) = a.module.smul(c, a.bracket(x, y))
} by {
    lie_bracket_smul_left(a, c, x, y)
}

/// The bracket is right-scalar-linear, as a rewrite-oriented endpoint.
theorem lie_bracket_smul_right_eq[R: CommRing, L: AddCommGroup](
    a: LieAlgebra[R, L], c: R, x: L, y: L
) {
    a.bracket(x, a.module.smul(c, y)) = a.module.smul(c, a.bracket(x, y))
} by {
    lie_bracket_smul_right(a, c, x, y)
}

/// The self bracket is zero, as a rewrite-oriented endpoint.
theorem lie_bracket_self_eq_zero[R: CommRing, L: AddCommGroup](a: LieAlgebra[R, L], x: L) {
    a.bracket(x, x) = L.0
} by {
    lie_bracket_self(a, x)
}

/// The bracket sends a zero left argument to zero, as a rewrite-oriented endpoint.
theorem lie_bracket_zero_left_eq[R: CommRing, L: AddCommGroup](a: LieAlgebra[R, L], x: L) {
    a.bracket(L.0, x) = L.0
} by {
    lie_bracket_zero_left(a, x)
}

/// The bracket sends a zero right argument to zero, as a rewrite-oriented endpoint.
theorem lie_bracket_zero_right_eq[R: CommRing, L: AddCommGroup](a: LieAlgebra[R, L], x: L) {
    a.bracket(x, L.0) = L.0
} by {
    lie_bracket_zero_right(a, x)
}

/// Adding zero on the right of the left argument does not change the bracket.
theorem lie_bracket_add_zero_left[R: CommRing, L: AddCommGroup](
    a: LieAlgebra[R, L], x: L, y: L
) {
    a.bracket(x + L.0, y) = a.bracket(x, y)
} by {
}

/// Adding zero on the left of the left argument does not change the bracket.
theorem lie_bracket_zero_add_left[R: CommRing, L: AddCommGroup](
    a: LieAlgebra[R, L], x: L, y: L
) {
    a.bracket(L.0 + x, y) = a.bracket(x, y)
} by {
}

/// Adding zero on the right of the right argument does not change the bracket.
theorem lie_bracket_add_zero_right[R: CommRing, L: AddCommGroup](
    a: LieAlgebra[R, L], x: L, y: L
) {
    a.bracket(x, y + L.0) = a.bracket(x, y)
} by {
}

/// Adding zero on the left of the right argument does not change the bracket.
theorem lie_bracket_zero_add_right[R: CommRing, L: AddCommGroup](
    a: LieAlgebra[R, L], x: L, y: L
) {
    a.bracket(x, L.0 + y) = a.bracket(x, y)
} by {
}

/// The bracket negates in the left argument, as a rewrite-oriented endpoint.
theorem lie_bracket_neg_left_eq[R: CommRing, L: AddCommGroup](
    a: LieAlgebra[R, L], x: L, y: L
) {
    a.bracket(-x, y) = -a.bracket(x, y)
} by {
    lie_bracket_neg_left(a, x, y)
}

/// The bracket negates in the right argument, as a rewrite-oriented endpoint.
theorem lie_bracket_neg_right_eq[R: CommRing, L: AddCommGroup](
    a: LieAlgebra[R, L], x: L, y: L
) {
    a.bracket(x, -y) = -a.bracket(x, y)
} by {
    lie_bracket_neg_right(a, x, y)
}

/// Reversed skew-symmetry: the negated swapped bracket rewrites back to the original bracket.
theorem lie_bracket_skew_rev[R: CommRing, L: AddCommGroup](a: LieAlgebra[R, L], x: L, y: L) {
    -a.bracket(y, x) = a.bracket(x, y)
} by {
    lie_bracket_skew(a, x, y)
}

/// Negating both arguments leaves the bracket unchanged.
theorem lie_bracket_neg_neg[R: CommRing, L: AddCommGroup](a: LieAlgebra[R, L], x: L, y: L) {
    a.bracket(-x, -y) = a.bracket(x, y)
} by {
    lie_bracket_neg_left(a, x, -y)
    lie_bracket_neg_right(a, x, y)
    inverse_inverse(a.bracket(x, y))
}

/// A scalar multiple of zero in the left argument gives a zero bracket.
theorem lie_bracket_smul_zero_left[R: CommRing, L: AddCommGroup](
    a: LieAlgebra[R, L], c: R, y: L
) {
    a.bracket(a.module.smul(c, L.0), y) = L.0
} by {
    module_smul_zero_right(a.module, c)
    lie_bracket_zero_left(a, y)
}

/// A scalar multiple of zero in the right argument gives a zero bracket.
theorem lie_bracket_smul_zero_right[R: CommRing, L: AddCommGroup](
    a: LieAlgebra[R, L], c: R, x: L
) {
    a.bracket(x, a.module.smul(c, L.0)) = L.0
} by {
    module_smul_zero_right(a.module, c)
    lie_bracket_zero_right(a, x)
}

/// A zero scalar in the left argument gives a zero bracket.
theorem lie_bracket_zero_smul_left[R: CommRing, L: AddCommGroup](
    a: LieAlgebra[R, L], x: L, y: L
) {
    a.bracket(a.module.smul(R.0, x), y) = L.0
} by {
    module_smul_zero_left(a.module, x)
    lie_bracket_zero_left(a, y)
}

/// A zero scalar in the right argument gives a zero bracket.
theorem lie_bracket_zero_smul_right[R: CommRing, L: AddCommGroup](
    a: LieAlgebra[R, L], x: L, y: L
) {
    a.bracket(x, a.module.smul(R.0, y)) = L.0
} by {
    module_smul_zero_left(a.module, y)
    lie_bracket_zero_right(a, x)
}

/// A cyclic rotation of the Jacobi identity.
theorem lie_bracket_jacobi_cyclic_left[R: CommRing, L: AddCommGroup](
    a: LieAlgebra[R, L], x: L, y: L, z: L
) {
    (a.bracket(y, a.bracket(z, x)) + a.bracket(z, a.bracket(x, y))) + a.bracket(x, a.bracket(y, z)) = L.0
} by {
    lie_bracket_jacobi(a, y, z, x)
}

/// The other cyclic rotation of the Jacobi identity.
theorem lie_bracket_jacobi_cyclic_right[R: CommRing, L: AddCommGroup](
    a: LieAlgebra[R, L], x: L, y: L, z: L
) {
    (a.bracket(z, a.bracket(x, y)) + a.bracket(x, a.bracket(y, z))) + a.bracket(y, a.bracket(z, x)) = L.0
} by {
    lie_bracket_jacobi(a, z, x, y)
}

/// The trivial (abelian) bracket sends every pair to zero.
let trivial_lie_bracket[R: CommRing, L: AddCommGroup]: (L, L) -> L =
    function(x: L, y: L) { L.0 }

/// The trivial bracket evaluates to zero.
theorem trivial_lie_bracket_zero[R: CommRing, L: AddCommGroup](x: L, y: L) {
    trivial_lie_bracket[R, L](x, y) = L.0
} by {
}

/// The trivial bracket forms a Lie algebra over any module structure.
theorem trivial_lie_bracket_is_lie_bracket[R: CommRing, L: AddCommGroup](m: Module[R, L]) {
    is_lie_bracket(m, trivial_lie_bracket[R, L])
} by {
    forall(x: L, y: L, z: L) {
        trivial_lie_bracket[R, L](x + y, z) = trivial_lie_bracket[R, L](x, z) + trivial_lie_bracket[R, L](y, z)
    }
    lie_bracket_add_left_constraint[R, L](trivial_lie_bracket[R, L])
    forall(x: L, y: L, z: L) {
        trivial_lie_bracket[R, L](x, y + z) = trivial_lie_bracket[R, L](x, y) + trivial_lie_bracket[R, L](x, z)
    }
    lie_bracket_add_right_constraint[R, L](trivial_lie_bracket[R, L])
    forall(c: R, x: L, y: L) {
        module_smul_zero_right(m, c)
        trivial_lie_bracket[R, L](m.smul(c, x), y) = m.smul(c, trivial_lie_bracket[R, L](x, y))
    }
    lie_bracket_smul_left_constraint(m, trivial_lie_bracket[R, L])
    forall(c: R, x: L, y: L) {
        module_smul_zero_right(m, c)
        trivial_lie_bracket[R, L](x, m.smul(c, y)) = m.smul(c, trivial_lie_bracket[R, L](x, y))
    }
    lie_bracket_smul_right_constraint(m, trivial_lie_bracket[R, L])
    forall(x: L) {
        trivial_lie_bracket[R, L](x, x) = L.0
    }
    lie_bracket_alternating_constraint[R, L](trivial_lie_bracket[R, L])
    forall(x: L, y: L, z: L) {
        (trivial_lie_bracket[R, L](x, trivial_lie_bracket[R, L](y, z)) + trivial_lie_bracket[R, L](y, trivial_lie_bracket[R, L](z, x))) + trivial_lie_bracket[R, L](z, trivial_lie_bracket[R, L](x, y)) = L.0
    }
    lie_bracket_jacobi_constraint[R, L](trivial_lie_bracket[R, L])
}

/// The abelian Lie algebra on an R-module: every bracket vanishes.
let abelian_lie_algebra[R: CommRing, L: AddCommGroup](m: Module[R, L]) -> result: LieAlgebra[R, L] satisfy {
    LieAlgebra[R, L].new(m, trivial_lie_bracket[R, L]) = Option.some(result)
} by {
    trivial_lie_bracket_is_lie_bracket[R, L](m)
}

/// The abelian Lie algebra has the trivial bracket.
theorem abelian_lie_algebra_bracket[R: CommRing, L: AddCommGroup](m: Module[R, L], x: L, y: L) {
    abelian_lie_algebra(m).bracket(x, y) = L.0
} by {
    abelian_lie_algebra(m).bracket = trivial_lie_bracket[R, L]
}

/// The abelian Lie algebra has the given module as its underlying module.
theorem abelian_lie_algebra_module[R: CommRing, L: AddCommGroup](m: Module[R, L]) {
    abelian_lie_algebra(m).module = m
}

/// The abelian Lie algebra has zero bracket, as a rewrite-oriented endpoint.
theorem abelian_lie_algebra_bracket_eq_zero[R: CommRing, L: AddCommGroup](
    m: Module[R, L], x: L, y: L
) {
    abelian_lie_algebra(m).bracket(x, y) = L.0
} by {
    abelian_lie_algebra_bracket(m, x, y)
}

/// The abelian Lie algebra has the given module, as a rewrite-oriented endpoint.
theorem abelian_lie_algebra_module_eq[R: CommRing, L: AddCommGroup](m: Module[R, L]) {
    abelian_lie_algebra(m).module = m
} by {
    abelian_lie_algebra_module(m)
}

/// True if a submodule of a Lie algebra's underlying module is closed under the Lie bracket.
define lie_subalgebra_bracket_constraint[R: CommRing, L: AddCommGroup](
    a: LieAlgebra[R, L], s: Submodule[R, L]
) -> Bool {
    forall(x: L, y: L) {
        s.contains(x) and s.contains(y) implies s.contains(a.bracket(x, y))
    }
}

/// True if a submodule is a Lie subalgebra of the given Lie algebra: its ambient module agrees with that of the Lie algebra and it is closed under the bracket.
define is_lie_subalgebra[R: CommRing, L: AddCommGroup](
    a: LieAlgebra[R, L], s: Submodule[R, L]
) -> Bool {
    s.carrier = a.module and lie_subalgebra_bracket_constraint(a, s)
}

/// A Lie subalgebra of a Lie algebra: a submodule of the underlying module that is closed under the Lie bracket.
structure LieSubalgebra[R: CommRing, L: AddCommGroup] {
    /// The ambient Lie algebra.
    ambient: LieAlgebra[R, L]
    /// The underlying submodule.
    submodule: Submodule[R, L]
} constraint {
    is_lie_subalgebra(ambient, submodule)
}

attributes LieSubalgebra[R: CommRing, L: AddCommGroup] {
    /// True if the given element belongs to this Lie subalgebra.
    define contains(self, x: L) -> Bool {
        self.submodule.contains(x)
    }
}

/// The underlying submodule of a Lie subalgebra shares the Lie algebra's module.
theorem lie_subalgebra_carrier[R: CommRing, L: AddCommGroup](s: LieSubalgebra[R, L]) {
    s.submodule.carrier = s.ambient.module
} by {
}

/// The bracket-closure constraint holds for every Lie subalgebra.
theorem lie_subalgebra_has_bracket_constraint[R: CommRing, L: AddCommGroup](
    s: LieSubalgebra[R, L]
) {
    lie_subalgebra_bracket_constraint(s.ambient, s.submodule)
} by {
}

/// The bracket-closure constraint expands to its forall body.
theorem lie_subalgebra_bracket_constraint_unfold[R: CommRing, L: AddCommGroup](
    a: LieAlgebra[R, L], s: Submodule[R, L]
) {
    lie_subalgebra_bracket_constraint(a, s) = forall(x: L, y: L) {
        s.contains(x) and s.contains(y) implies s.contains(a.bracket(x, y))
    }
}

/// A Lie subalgebra is closed under the Lie bracket.
theorem lie_subalgebra_bracket[R: CommRing, L: AddCommGroup](
    s: LieSubalgebra[R, L], x: L, y: L
) {
    s.submodule.contains(x) and s.submodule.contains(y)
        implies s.submodule.contains(s.ambient.bracket(x, y))
} by {
    if s.submodule.contains(x) and s.submodule.contains(y) {
        lie_subalgebra_has_bracket_constraint(s)
        lie_subalgebra_bracket_constraint(s.ambient, s.submodule) = forall(u: L, v: L) {
            s.submodule.contains(u) and s.submodule.contains(v) implies s.submodule.contains(s.ambient.bracket(u, v))
        }
        s.submodule.contains(s.ambient.bracket(x, y))
    }
}

/// Membership in a Lie subalgebra is membership in its underlying submodule.
theorem lie_subalgebra_contains_iff_submodule_contains[R: CommRing, L: AddCommGroup](
    s: LieSubalgebra[R, L], x: L
) {
    s.contains(x) = s.submodule.contains(x)
} by {
}

/// The underlying submodule carrier is the ambient Lie algebra module.
theorem lie_subalgebra_carrier_eq_ambient_module[R: CommRing, L: AddCommGroup](s: LieSubalgebra[R, L]) {
    s.submodule.carrier = s.ambient.module
} by {
    lie_subalgebra_carrier(s)
}

/// A Lie subalgebra contains zero.
theorem lie_subalgebra_contains_zero[R: CommRing, L: AddCommGroup](s: LieSubalgebra[R, L]) {
    s.contains(L.0)
} by {
    submodule_contains_zero(s.submodule)
}

/// A Lie subalgebra is closed under addition.
theorem lie_subalgebra_contains_add[R: CommRing, L: AddCommGroup](
    s: LieSubalgebra[R, L], x: L, y: L
) {
    s.contains(x) and s.contains(y) implies s.contains(x + y)
} by {
    if s.contains(x) and s.contains(y) {
        lie_subalgebra_contains_iff_submodule_contains(s, x)
        lie_subalgebra_contains_iff_submodule_contains(s, y)
        lie_subalgebra_contains_iff_submodule_contains(s, x + y)
        submodule_contains_add(s.submodule, x, y)
        s.contains(x + y)
    }
}

/// A Lie subalgebra is closed under additive negation.
theorem lie_subalgebra_contains_neg[R: CommRing, L: AddCommGroup](
    s: LieSubalgebra[R, L], x: L
) {
    s.contains(x) implies s.contains(-x)
} by {
    if s.contains(x) {
        lie_subalgebra_contains_iff_submodule_contains(s, x)
        lie_subalgebra_contains_iff_submodule_contains(s, -x)
        submodule_contains_neg(s.submodule, x)
        s.contains(-x)
    }
}

/// A Lie subalgebra is closed under scalar multiplication in its underlying submodule.
theorem lie_subalgebra_contains_smul[R: CommRing, L: AddCommGroup](
    s: LieSubalgebra[R, L], r: R, x: L
) {
    s.contains(x) implies s.contains(s.submodule.carrier.smul(r, x))
} by {
    if s.contains(x) {
        lie_subalgebra_contains_iff_submodule_contains(s, x)
        lie_subalgebra_contains_iff_submodule_contains(s, s.submodule.carrier.smul(r, x))
        submodule_contains_smul(s.submodule, r, x)
        s.contains(s.submodule.carrier.smul(r, x))
    }
}

/// A Lie subalgebra is closed under its Lie bracket, using the `contains` projection.
theorem lie_subalgebra_bracket_closed[R: CommRing, L: AddCommGroup](
    s: LieSubalgebra[R, L], x: L, y: L
) {
    s.contains(x) and s.contains(y) implies s.contains(s.ambient.bracket(x, y))
} by {
    if s.contains(x) and s.contains(y) {
        lie_subalgebra_contains_iff_submodule_contains(s, x)
        lie_subalgebra_contains_iff_submodule_contains(s, y)
        lie_subalgebra_contains_iff_submodule_contains(s, s.ambient.bracket(x, y))
        lie_subalgebra_bracket(s, x, y)
        s.contains(s.ambient.bracket(x, y))
    }
}

// TODO: lie_subalgebra_ext, full_submodule_is_lie_subalgebra, and lie_subalgebra_top
// were dropped in the Submodule-based refactor because their natural Acorn proofs
// time out at the default 5s search budget. Re-add them once we have stable proof
// hints for goals of the shape `lie_subalgebra_bracket_constraint(a, s)` over the
// bundled submodule predicate.

/// True if a submodule of a Lie algebra's underlying module absorbs the bracket from the left.
define lie_ideal_bracket_constraint[R: CommRing, L: AddCommGroup](
    a: LieAlgebra[R, L], s: Submodule[R, L]
) -> Bool {
    forall(x: L, y: L) {
        s.contains(y) implies s.contains(a.bracket(x, y))
    }
}

/// True if a submodule is a Lie ideal of the given Lie algebra: its ambient module agrees with that of the Lie algebra and it absorbs the bracket.
define is_lie_ideal[R: CommRing, L: AddCommGroup](
    a: LieAlgebra[R, L], s: Submodule[R, L]
) -> Bool {
    s.carrier = a.module and lie_ideal_bracket_constraint(a, s)
}

/// A Lie ideal of a Lie algebra: a submodule of the underlying module that absorbs the bracket with arbitrary ambient elements.
structure LieIdeal[R: CommRing, L: AddCommGroup] {
    /// The ambient Lie algebra.
    ambient: LieAlgebra[R, L]
    /// The underlying submodule.
    submodule: Submodule[R, L]
} constraint {
    is_lie_ideal(ambient, submodule)
}

attributes LieIdeal[R: CommRing, L: AddCommGroup] {
    /// True if the given element belongs to this Lie ideal.
    define contains(self, x: L) -> Bool {
        self.submodule.contains(x)
    }
}

/// The underlying submodule of a Lie ideal shares the Lie algebra's module.
theorem lie_ideal_carrier[R: CommRing, L: AddCommGroup](i: LieIdeal[R, L]) {
    i.submodule.carrier = i.ambient.module
} by {
}

/// The bracket-absorption constraint holds for every Lie ideal.
theorem lie_ideal_has_bracket_constraint[R: CommRing, L: AddCommGroup](
    i: LieIdeal[R, L]
) {
    lie_ideal_bracket_constraint(i.ambient, i.submodule)
} by {
}

/// The bracket-absorption constraint expands to its quantified membership statement.
theorem lie_ideal_bracket_constraint_unfold[R: CommRing, L: AddCommGroup](
    a: LieAlgebra[R, L], s: Submodule[R, L]
) {
    lie_ideal_bracket_constraint(a, s) = forall(x: L, y: L) {
        s.contains(y) implies s.contains(a.bracket(x, y))
    }
}

/// Membership in a Lie ideal is membership in its underlying submodule.
theorem lie_ideal_contains_iff_submodule_contains[R: CommRing, L: AddCommGroup](
    i: LieIdeal[R, L], x: L
) {
    i.contains(x) = i.submodule.contains(x)
} by {
}

/// The underlying ideal submodule carrier is the ambient Lie algebra module.
theorem lie_ideal_carrier_eq_ambient_module[R: CommRing, L: AddCommGroup](i: LieIdeal[R, L]) {
    i.submodule.carrier = i.ambient.module
} by {
    lie_ideal_carrier(i)
}

/// A Lie ideal contains zero.
theorem lie_ideal_contains_zero[R: CommRing, L: AddCommGroup](i: LieIdeal[R, L]) {
    i.contains(L.0)
} by {
    submodule_contains_zero(i.submodule)
}

/// A Lie ideal is closed under addition as a submodule.
theorem lie_ideal_contains_add[R: CommRing, L: AddCommGroup](i: LieIdeal[R, L], x: L, y: L) {
    i.contains(x) and i.contains(y) implies i.contains(x + y)
} by {
    if i.contains(x) and i.contains(y) {
        lie_ideal_contains_iff_submodule_contains(i, x)
        lie_ideal_contains_iff_submodule_contains(i, y)
        lie_ideal_contains_iff_submodule_contains(i, x + y)
        submodule_contains_add(i.submodule, x, y)
        i.contains(x + y)
    }
}

/// A Lie ideal is closed under additive negation as a submodule.
theorem lie_ideal_contains_neg[R: CommRing, L: AddCommGroup](i: LieIdeal[R, L], x: L) {
    i.contains(x) implies i.contains(-x)
} by {
    if i.contains(x) {
        lie_ideal_contains_iff_submodule_contains(i, x)
        lie_ideal_contains_iff_submodule_contains(i, -x)
        submodule_contains_neg(i.submodule, x)
        i.contains(-x)
    }
}

/// A Lie ideal is closed under scalar multiplication in its underlying submodule.
theorem lie_ideal_contains_smul[R: CommRing, L: AddCommGroup](i: LieIdeal[R, L], r: R, x: L) {
    i.contains(x) implies i.contains(i.submodule.carrier.smul(r, x))
} by {
    if i.contains(x) {
        lie_ideal_contains_iff_submodule_contains(i, x)
        lie_ideal_contains_iff_submodule_contains(i, i.submodule.carrier.smul(r, x))
        submodule_contains_smul(i.submodule, r, x)
        i.contains(i.submodule.carrier.smul(r, x))
    }
}

// TODO: lie_ideal_bracket_left and lie_ideal_bracket_right.
// A minimal bundled-constraint training example lives in
// hard_problems/lie_ideal_bracket_left.ac.

// TODO: lie_ideal_ext, lie_ideal_is_lie_subalgebra, zero_submodule_is_lie_ideal,
// and lie_zero_ideal were dropped in the Submodule-based refactor for the same
// reason as the subalgebra versions above: the natural Acorn proofs do not
// converge under the default 5s search budget.
