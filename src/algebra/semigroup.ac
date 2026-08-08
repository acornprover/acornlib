from algebra.mul import Mul

/// The default semigroup uses the multiplication operator..
/// For an additive semigroup, see add_semigroup.ac.
typeclass S: Semigroup extends Mul {
    /// The multiplication operation must be associative: `(a * b) * c = a * (b * c)`.
    mul_associative(a: S, b: S, c: S) {
        a * (b * c) = (a * b) * c
    }
}

/// Scalar multiplication of a function: multiplies a constant by the result of a function.
define mul_fn[T, S: Semigroup](c: S, f: T -> S, t: T) -> S {
    c * f(t)
}