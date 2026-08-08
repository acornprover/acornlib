from comm_ring import CommRing
from data.basic.set import indexed_union, indexed_union_contains_of_contains,
    indexed_union_contains_witness
from algebra.ring.ideal import Ideal, bundled_ideal_subset, ideal_as_set_contains_eq,
    ideal_closure_contains_of_set_contains,
    ideal_closure_le_iff_set_subset, ideal_family_as_set, set_subset_ideal

/// The indexed sum of a family of ideals: the ideal generated by the union of their underlying sets.
define ideal_indexed_sum[R: CommRing, I](family: I -> Ideal[R]) -> Ideal[R] {
    Ideal[R].closure(indexed_union(ideal_family_as_set(family)))
}

/// Each member ideal is contained in the indexed sum.
theorem ideal_indexed_sum_contains_family[R: CommRing, I](family: I -> Ideal[R], i: I, x: R) {
    family(i).contains(x) implies ideal_indexed_sum(family).contains(x)
} by {
    if family(i).contains(x) {
        ideal_as_set_contains_eq(family(i), x)
        indexed_union_contains_of_contains(ideal_family_as_set(family), i, x)
        indexed_union(ideal_family_as_set(family)).contains(x)
        ideal_closure_contains_of_set_contains(indexed_union(ideal_family_as_set(family)), x)
        ideal_indexed_sum(family).contains(x)
    }
}

/// Each member ideal is bundled-contained in the indexed sum.
theorem ideal_indexed_sum_family_subset[R: CommRing, I](family: I -> Ideal[R], i: I) {
    bundled_ideal_subset(family(i), ideal_indexed_sum(family))
} by {
    forall(x: R) {
        if family(i).contains(x) {
            ideal_indexed_sum_contains_family(family, i, x)
            ideal_indexed_sum(family).contains(x)
        }
    }
}

/// The universal property eliminates membership from a family member into any upper bound.
theorem ideal_indexed_sum_subset_elim[R: CommRing, I](family: I -> Ideal[R], c: Ideal[R], i: I, x: R) {
    bundled_ideal_subset(ideal_indexed_sum(family), c) and family(i).contains(x) implies c.contains(x)
} by {
    if bundled_ideal_subset(ideal_indexed_sum(family), c) and family(i).contains(x) {
        ideal_indexed_sum_contains_family(family, i, x)
        bundled_ideal_subset(ideal_indexed_sum(family), c) = forall(y: R) {
            ideal_indexed_sum(family).contains(y) implies c.contains(y)
        }
        c.contains(x)
    }
}

/// The indexed sum is contained in an ideal when every member ideal is contained in that ideal.
theorem ideal_indexed_sum_subset_of_family_subset[R: CommRing, I](family: I -> Ideal[R], c: Ideal[R]) {
    (forall(i: I) { bundled_ideal_subset(family(i), c) })
        implies bundled_ideal_subset(ideal_indexed_sum(family), c)
} by {
    if forall(i: I) { bundled_ideal_subset(family(i), c) } {
        forall(x: R) {
            if indexed_union(ideal_family_as_set(family)).contains(x) {
                indexed_union_contains_witness(ideal_family_as_set(family), x)
                let i: I satisfy { ideal_family_as_set(family)(i).contains(x) }
                ideal_as_set_contains_eq(family(i), x)
                family(i).contains(x)
                bundled_ideal_subset(family(i), c) = forall(y: R) {
                    family(i).contains(y) implies c.contains(y)
                }
                c.contains(x)
            }
        }
        set_subset_ideal(indexed_union(ideal_family_as_set(family)), c)
        ideal_closure_le_iff_set_subset(indexed_union(ideal_family_as_set(family)), c)
        bundled_ideal_subset(ideal_indexed_sum(family), c)
    }
}

/// The indexed sum satisfies the expected least-upper-bound universal property.
theorem ideal_indexed_sum_subset_iff[R: CommRing, I](family: I -> Ideal[R], c: Ideal[R]) {
    bundled_ideal_subset(ideal_indexed_sum(family), c) =
        forall(i: I) { bundled_ideal_subset(family(i), c) }
} by {
    if bundled_ideal_subset(ideal_indexed_sum(family), c) {
        ideal_closure_le_iff_set_subset(indexed_union(ideal_family_as_set(family)), c)
        set_subset_ideal(indexed_union(ideal_family_as_set(family)), c)
        forall(i: I) {
            forall(x: R) {
                if family(i).contains(x) {
                    ideal_as_set_contains_eq(family(i), x)
                    indexed_union_contains_of_contains(ideal_family_as_set(family), i, x)
                    indexed_union(ideal_family_as_set(family)).contains(x)
                    set_subset_ideal(indexed_union(ideal_family_as_set(family)), c) = forall(y: R) {
                        indexed_union(ideal_family_as_set(family)).contains(y) implies c.contains(y)
                    }
                    c.contains(x)
                }
            }
            bundled_ideal_subset(family(i), c)
        }
    }
    if forall(i: I) { bundled_ideal_subset(family(i), c) } {
        ideal_indexed_sum_subset_of_family_subset(family, c)
        bundled_ideal_subset(ideal_indexed_sum(family), c)
    }
}

/// Indexed sums are monotone with respect to pointwise containment of families.
theorem ideal_indexed_sum_monotone[R: CommRing, I](family: I -> Ideal[R], g: I -> Ideal[R]) {
    (forall(i: I) { bundled_ideal_subset(family(i), g(i)) })
        implies bundled_ideal_subset(ideal_indexed_sum(family), ideal_indexed_sum(g))
} by {
    if forall(i: I) { bundled_ideal_subset(family(i), g(i)) } {
        forall(i: I) {
            forall(x: R) {
                if family(i).contains(x) {
                    bundled_ideal_subset(family(i), g(i)) = forall(y: R) {
                        family(i).contains(y) implies g(i).contains(y)
                    }
                    ideal_indexed_sum_contains_family(g, i, x)
                    ideal_indexed_sum(g).contains(x)
                }
            }
            bundled_ideal_subset(family(i), ideal_indexed_sum(g))
        }
        ideal_indexed_sum_subset_of_family_subset(family, ideal_indexed_sum(g))
        bundled_ideal_subset(ideal_indexed_sum(family), ideal_indexed_sum(g))
    }
}
