/// The quotient ring `R/I` of a commutative ring by an ideal, packaged as a
/// bundled type together with its ring structure, the canonical projection, and
/// the first isomorphism theorem for the quotient by a ring-homomorphism kernel.
///
/// The underlying quotient machinery lives in `algebra/ring/ideal_quotient.ac`
/// (operations on `QuotientOver[R]`, well-definedness, and the ring laws on
/// representatives) and `data/basic/quotient_algebra.ac` (kernel quotients of
/// ring homomorphisms).  This file bundles the ideal quotient into a type whose
/// elements carry the ideal quotient relation, installs the `Ring` instance,
/// packages the canonical projection as a `RingHom`, and connects the ideal
/// quotient by the kernel of a homomorphism with the kernel quotient, yielding
/// the first isomorphism theorem.

from comm_ring import CommRing
from algebra.ring.ideal import Ideal, ring_hom_kernel, ring_hom_kernel_contains_eq
from algebra.ring.ring_hom import RingHom
from algebra.hom_kernel_injective import ring_hom_eq_iff_sub_zero
from algebra.ring.ideal_quotient import ideal_quotient_relation, ideal_quotient_relation_rel,
    ideal_quotient_project, ideal_quotient_project_eq_mk,
    ideal_quotient_add, ideal_quotient_add_projection,
    ideal_quotient_mul, ideal_quotient_mul_projection,
    ideal_quotient_neg, ideal_quotient_neg_projection,
    ideal_quotient_zero, ideal_quotient_zero_projection,
    ideal_quotient_one, ideal_quotient_one_projection,
    ideal_quotient_project_eq_iff_contains_sub
from data.basic.equivalence import QuotientOver, quotient_over_mk, quotient_over_mk_fields,
    quotient_over_mk_surjective, quotient_over_binary_op_at, quotient_over_binary_op_at_has_representatives,
    quotient_over_lift_at, quotient_over_lift_at_has_representative,
    quotient_over_eq_of_fields_eq, quotient_relation_ext,
    kernel_relation, kernel_quotient_relation, kernel_quotient_relation_rel
from data.basic.set import equivalence_class
from data.basic.quotient_algebra import ideal_quotient_rel, ideal_quotient_rel_eq_contains_sub,
    ring_hom_kernel_quotient_project, ring_hom_kernel_quotient_project_eq_mk,
    ring_hom_kernel_quotient_induced, ring_hom_kernel_quotient_induced_project,
    ring_hom_kernel_quotient_induced_injective
from data.basic.functions import binary_function_extensionality
from algebra.add import Add
from algebra.zero import Zero
from algebra.neg import Neg
from algebra.mul import Mul
from algebra.one import One
from algebra.add_semigroup import AddSemigroup
from algebra.add_comm_semigroup import AddCommSemigroup
from algebra.add_monoid import AddMonoid
from algebra.add_comm_monoid import AddCommMonoid
from algebra.add_group import AddGroup
from algebra.add_comm_group import AddCommGroup
from algebra.semigroup import Semigroup
from algebra.comm_semigroup import CommSemigroup
from algebra.monoid.monoid import Monoid
from algebra.comm_monoid import CommMonoid
from semiring import Semiring
from algebra.ring.ring import Ring

/// A quotient ring element: a quotient-over element of `R` whose stored
/// equivalence relation is the ideal quotient relation of `i`.
structure IdealQuotient[R: CommRing, i: Ideal[R]] {
    /// The underlying quotient-over element of `R`.
    val: QuotientOver[R]
} constraint {
    val.qrel = ideal_quotient_relation(i)
}

/// Construction of a quotient-ring element remembers its underlying value.
theorem ideal_quotient_type_new_round_trip[R: CommRing](
    i: Ideal[R], p: QuotientOver[R], z: IdealQuotient[R, i]
) {
    IdealQuotient[R, i].new(p) = Option.some(z) implies z.val = p
}

/// Every quotient-ring element is reconstructed from its underlying value.
theorem ideal_quotient_type_new_self[R: CommRing](i: Ideal[R], z: IdealQuotient[R, i]) {
    IdealQuotient[R, i].new(z.val) = Option.some(z)
}

/// The canonical projection lies over the ideal quotient relation.
theorem ideal_quotient_project_qrel[R: CommRing](i: Ideal[R], a: R) {
    ideal_quotient_project(i, a).qrel = ideal_quotient_relation(i)
} by {
    ideal_quotient_project_eq_mk(i, a)
    ideal_quotient_project(i, a).qrel = quotient_over_mk(ideal_quotient_relation(i), a).qrel
    quotient_over_mk_fields(ideal_quotient_relation(i), a)
    quotient_over_mk(ideal_quotient_relation(i), a).qrel = ideal_quotient_relation(i)
}

/// The quotient-ring element represented by `a`.
let ideal_quotient_mk_type[R: CommRing](i: Ideal[R], a: R) -> result: IdealQuotient[R, i] satisfy {
    IdealQuotient[R, i].new(ideal_quotient_project(i, a)) = Option.some(result)
}

/// The underlying value of a quotient-ring projection is the canonical projection.
theorem ideal_quotient_mk_type_val[R: CommRing](i: Ideal[R], a: R) {
    ideal_quotient_mk_type(i, a).val = ideal_quotient_project(i, a)
} by {
    ideal_quotient_type_new_round_trip(i, ideal_quotient_project(i, a), ideal_quotient_mk_type(i, a))
}

/// Wrapped addition of quotient-ring elements lies over the ideal quotient relation.
theorem ideal_quotient_type_add_qrel[R: CommRing](i: Ideal[R], x: IdealQuotient[R, i], y: IdealQuotient[R, i]) {
    ideal_quotient_add(i, x.val, y.val).qrel = ideal_quotient_relation(i)
} by {
    quotient_over_binary_op_at_has_representatives(R.add, x.val, y.val)
    let (a: R, b: R) satisfy {
        quotient_over_mk(x.val.qrel, a) = x.val and
        quotient_over_mk(y.val.qrel, b) = y.val and
        quotient_over_binary_op_at(R.add, x.val, y.val) = quotient_over_mk(x.val.qrel, a + b)
    }
    x.val.qrel = ideal_quotient_relation(i)
    ideal_quotient_add(i, x.val, y.val) =
        quotient_over_binary_op_at(R.add, x.val, y.val)
    ideal_quotient_add(i, x.val, y.val) = quotient_over_mk(x.val.qrel, a + b)
    ideal_quotient_add(i, x.val, y.val).qrel =
        quotient_over_mk(x.val.qrel, a + b).qrel
    quotient_over_mk_fields(x.val.qrel, a + b)
    quotient_over_mk(x.val.qrel, a + b).qrel = x.val.qrel
    ideal_quotient_add(i, x.val, y.val).qrel = x.val.qrel
    ideal_quotient_add(i, x.val, y.val).qrel = ideal_quotient_relation(i)
}

/// Addition on the quotient ring `R/I`.
let ideal_quotient_type_add[R: CommRing](i: Ideal[R], x: IdealQuotient[R, i], y: IdealQuotient[R, i]) -> result: IdealQuotient[R, i] satisfy {
    IdealQuotient[R, i].new(ideal_quotient_add(i, x.val, y.val)) = Option.some(result)
}

/// Wrapped multiplication of quotient-ring elements lies over the ideal quotient relation.
theorem ideal_quotient_type_mul_qrel[R: CommRing](i: Ideal[R], x: IdealQuotient[R, i], y: IdealQuotient[R, i]) {
    ideal_quotient_mul(i, x.val, y.val).qrel = ideal_quotient_relation(i)
} by {
    quotient_over_binary_op_at_has_representatives(R.mul, x.val, y.val)
    let (a: R, b: R) satisfy {
        quotient_over_mk(x.val.qrel, a) = x.val and
        quotient_over_mk(y.val.qrel, b) = y.val and
        quotient_over_binary_op_at(R.mul, x.val, y.val) = quotient_over_mk(x.val.qrel, a * b)
    }
    x.val.qrel = ideal_quotient_relation(i)
    ideal_quotient_mul(i, x.val, y.val) =
        quotient_over_binary_op_at(R.mul, x.val, y.val)
    ideal_quotient_mul(i, x.val, y.val) = quotient_over_mk(x.val.qrel, a * b)
    ideal_quotient_mul(i, x.val, y.val).qrel =
        quotient_over_mk(x.val.qrel, a * b).qrel
    quotient_over_mk_fields(x.val.qrel, a * b)
    quotient_over_mk(x.val.qrel, a * b).qrel = x.val.qrel
    ideal_quotient_mul(i, x.val, y.val).qrel = x.val.qrel
    ideal_quotient_mul(i, x.val, y.val).qrel = ideal_quotient_relation(i)
}

/// Multiplication on the quotient ring `R/I`.
let ideal_quotient_type_mul[R: CommRing](i: Ideal[R], x: IdealQuotient[R, i], y: IdealQuotient[R, i]) -> result: IdealQuotient[R, i] satisfy {
    IdealQuotient[R, i].new(ideal_quotient_mul(i, x.val, y.val)) = Option.some(result)
}

/// Wrapped negation of quotient-ring elements lies over the ideal quotient relation.
theorem ideal_quotient_type_neg_qrel[R: CommRing](i: Ideal[R], x: IdealQuotient[R, i]) {
    ideal_quotient_neg(i, x.val).qrel = ideal_quotient_relation(i)
} by {
    quotient_over_lift_at_has_representative(ideal_quotient_relation(i), R.neg, x.val)
    let a: R satisfy {
        quotient_over_mk(x.val.qrel, a) = x.val and
        quotient_over_lift_at(ideal_quotient_relation(i), R.neg, x.val) =
            quotient_over_mk(ideal_quotient_relation(i), R.neg(a))
    }
    ideal_quotient_neg(i, x.val) =
        quotient_over_lift_at(ideal_quotient_relation(i), R.neg, x.val)
    ideal_quotient_neg(i, x.val) = quotient_over_mk(ideal_quotient_relation(i), R.neg(a))
    quotient_over_mk_fields(ideal_quotient_relation(i), R.neg(a))
    quotient_over_mk(ideal_quotient_relation(i), R.neg(a)).qrel = ideal_quotient_relation(i)
    ideal_quotient_neg(i, x.val).qrel = ideal_quotient_relation(i)
}

/// Negation on the quotient ring `R/I`.
let ideal_quotient_type_neg[R: CommRing](i: Ideal[R], x: IdealQuotient[R, i]) -> result: IdealQuotient[R, i] satisfy {
    IdealQuotient[R, i].new(ideal_quotient_neg(i, x.val)) = Option.some(result)
}

/// The zero element of the quotient ring lies over the ideal quotient relation.
theorem ideal_quotient_type_zero_qrel[R: CommRing](i: Ideal[R]) {
    ideal_quotient_zero(i).qrel = ideal_quotient_relation(i)
} by {
    ideal_quotient_zero_projection(i)
    quotient_over_mk_fields(ideal_quotient_relation(i), R.0)
    quotient_over_mk(ideal_quotient_relation(i), R.0).qrel = ideal_quotient_relation(i)
    ideal_quotient_zero(i).qrel = ideal_quotient_relation(i)
}

/// The zero element of the quotient ring `R/I`.
let ideal_quotient_type_zero[R: CommRing](i: Ideal[R]) -> result: IdealQuotient[R, i] satisfy {
    IdealQuotient[R, i].new(ideal_quotient_zero(i)) = Option.some(result)
}

/// The one element of the quotient ring lies over the ideal quotient relation.
theorem ideal_quotient_type_one_qrel[R: CommRing](i: Ideal[R]) {
    ideal_quotient_one(i).qrel = ideal_quotient_relation(i)
} by {
    ideal_quotient_one_projection(i)
    quotient_over_mk_fields(ideal_quotient_relation(i), R.1)
    quotient_over_mk(ideal_quotient_relation(i), R.1).qrel = ideal_quotient_relation(i)
    ideal_quotient_one(i).qrel = ideal_quotient_relation(i)
}

/// The one element of the quotient ring `R/I`.
let ideal_quotient_type_one[R: CommRing](i: Ideal[R]) -> result: IdealQuotient[R, i] satisfy {
    IdealQuotient[R, i].new(ideal_quotient_one(i)) = Option.some(result)
}

/// The stored relation of a quotient-ring element is the ideal quotient relation.
theorem ideal_quotient_type_qrel[R: CommRing](i: Ideal[R], x: IdealQuotient[R, i]) {
    x.val.qrel = ideal_quotient_relation(i)
} by {
}

/// Extensionality for the quotient ring type.
theorem ideal_quotient_type_ext[R: CommRing](i: Ideal[R], x: IdealQuotient[R, i], y: IdealQuotient[R, i]) {
    x.val = y.val implies x = y
} by {
    if x.val = y.val {
        ideal_quotient_type_new_self(i, x)
        ideal_quotient_type_new_self(i, y)
        Option.some(x) = Option.some(y)
        x = y
    }
}

/// Every quotient-ring element is a projection of a representative.
theorem ideal_quotient_mk_type_surjective[R: CommRing](i: Ideal[R], x: IdealQuotient[R, i]) {
    exists(a: R) {
        ideal_quotient_mk_type(i, a) = x
    }
} by {
    quotient_over_mk_surjective(x.val)
    let a: R satisfy {
        quotient_over_mk(x.val.qrel, a) = x.val
    }
    ideal_quotient_type_qrel(i, x)
    quotient_over_mk(ideal_quotient_relation(i), a) = x.val
    ideal_quotient_project_eq_mk(i, a)
    ideal_quotient_project(i, a) = quotient_over_mk(ideal_quotient_relation(i), a)
    ideal_quotient_mk_type_val(i, a)
    ideal_quotient_mk_type(i, a).val = ideal_quotient_project(i, a)
    ideal_quotient_mk_type(i, a).val = x.val
    ideal_quotient_type_ext(i, ideal_quotient_mk_type(i, a), x)
    ideal_quotient_mk_type(i, a) = x
    exists(a0: R) {
        ideal_quotient_mk_type(i, a0) = x
    }
}

/// Wrapped addition of quotient-ring elements is addition of underlying values.
theorem ideal_quotient_type_add_val[R: CommRing](i: Ideal[R], x: IdealQuotient[R, i], y: IdealQuotient[R, i]) {
    ideal_quotient_type_add(i, x, y).val = ideal_quotient_add(i, x.val, y.val)
} by {
    ideal_quotient_type_new_round_trip(i, ideal_quotient_add(i, x.val, y.val),
        ideal_quotient_type_add(i, x, y))
}

/// Wrapped multiplication of quotient-ring elements is multiplication of underlying values.
theorem ideal_quotient_type_mul_val[R: CommRing](i: Ideal[R], x: IdealQuotient[R, i], y: IdealQuotient[R, i]) {
    ideal_quotient_type_mul(i, x, y).val = ideal_quotient_mul(i, x.val, y.val)
} by {
    ideal_quotient_type_new_round_trip(i, ideal_quotient_mul(i, x.val, y.val),
        ideal_quotient_type_mul(i, x, y))
}

/// Wrapped negation of quotient-ring elements is negation of underlying values.
theorem ideal_quotient_type_neg_val[R: CommRing](i: Ideal[R], x: IdealQuotient[R, i]) {
    ideal_quotient_type_neg(i, x).val = ideal_quotient_neg(i, x.val)
} by {
    ideal_quotient_type_new_round_trip(i, ideal_quotient_neg(i, x.val),
        ideal_quotient_type_neg(i, x))
}

/// The wrapped zero is the projection of the ring zero.
theorem ideal_quotient_type_zero_val[R: CommRing](i: Ideal[R]) {
    ideal_quotient_type_zero(i).val = ideal_quotient_zero(i)
} by {
    ideal_quotient_type_new_round_trip(i, ideal_quotient_zero(i), ideal_quotient_type_zero(i))
}

/// The wrapped one is the projection of the ring one.
theorem ideal_quotient_type_one_val[R: CommRing](i: Ideal[R]) {
    ideal_quotient_type_one(i).val = ideal_quotient_one(i)
} by {
    ideal_quotient_type_new_round_trip(i, ideal_quotient_one(i), ideal_quotient_type_one(i))
}

/// The typeclass operations on the quotient ring.
attributes IdealQuotient[R: CommRing, i: Ideal[R]] {
    /// Addition of quotient-ring elements.
    define add(self, other: IdealQuotient[R, i]) -> IdealQuotient[R, i] {
        ideal_quotient_type_add(i, self, other)
    }

    /// Multiplication of quotient-ring elements.
    define mul(self, other: IdealQuotient[R, i]) -> IdealQuotient[R, i] {
        ideal_quotient_type_mul(i, self, other)
    }

    /// Negation of quotient-ring elements.
    define neg(self) -> IdealQuotient[R, i] {
        ideal_quotient_type_neg(i, self)
    }

    /// The zero of the quotient ring.
    let zero: IdealQuotient[R, i] = ideal_quotient_type_zero(i)

    /// The one of the quotient ring.
    let one: IdealQuotient[R, i] = ideal_quotient_type_one(i)
}

/// Quotient-ring elements have addition.
instance IdealQuotient[R: CommRing, i: Ideal[R]]: Add {
    let add = IdealQuotient[R, i].add
}

/// Quotient-ring elements have a zero element.
instance IdealQuotient[R: CommRing, i: Ideal[R]]: Zero {
    let 0 = IdealQuotient[R, i].zero
}

/// Quotient-ring elements have additive inverses.
instance IdealQuotient[R: CommRing, i: Ideal[R]]: Neg {
    let neg = IdealQuotient[R, i].neg
}

/// Quotient-ring elements have multiplication.
instance IdealQuotient[R: CommRing, i: Ideal[R]]: Mul {
    let mul = IdealQuotient[R, i].mul
}

/// Quotient-ring elements have a one element.
instance IdealQuotient[R: CommRing, i: Ideal[R]]: One {
    let 1 = IdealQuotient[R, i].one
}

/// Typeclass addition on the quotient ring agrees with wrapped addition.
theorem ideal_quotient_typeclass_add_eq[R: CommRing](i: Ideal[R], x: IdealQuotient[R, i], y: IdealQuotient[R, i]) {
    x + y = ideal_quotient_type_add(i, x, y)
} by {
    x + y = ideal_quotient_type_add(i, x, y)
}

/// Typeclass multiplication on the quotient ring agrees with wrapped multiplication.
theorem ideal_quotient_typeclass_mul_eq[R: CommRing](i: Ideal[R], x: IdealQuotient[R, i], y: IdealQuotient[R, i]) {
    x * y = ideal_quotient_type_mul(i, x, y)
} by {
    x * y = ideal_quotient_type_mul(i, x, y)
}

/// Typeclass negation on the quotient ring agrees with wrapped negation.
theorem ideal_quotient_typeclass_neg_eq[R: CommRing](i: Ideal[R], x: IdealQuotient[R, i]) {
    -x = ideal_quotient_type_neg(i, x)
} by {
    -x = ideal_quotient_type_neg(i, x)
}

/// Typeclass zero on the quotient ring agrees with the wrapped zero.
theorem ideal_quotient_typeclass_zero_eq[R: CommRing](i: Ideal[R]) {
    Zero.0[IdealQuotient[R, i]] = ideal_quotient_type_zero(i)
} by {
    Zero.0[IdealQuotient[R, i]] = ideal_quotient_type_zero(i)
}

/// Typeclass one on the quotient ring agrees with the wrapped one.
theorem ideal_quotient_typeclass_one_eq[R: CommRing](i: Ideal[R]) {
    One.1[IdealQuotient[R, i]] = ideal_quotient_type_one(i)
} by {
    One.1[IdealQuotient[R, i]] = ideal_quotient_type_one(i)
}

/// Wrapped addition agrees with addition of representatives.
theorem ideal_quotient_type_add_mk[R: CommRing](i: Ideal[R], a: R, b: R) {
    ideal_quotient_type_add(i, ideal_quotient_mk_type(i, a), ideal_quotient_mk_type(i, b)) =
        ideal_quotient_mk_type(i, a + b)
} by {
    ideal_quotient_type_add_val(i, ideal_quotient_mk_type(i, a), ideal_quotient_mk_type(i, b))
    ideal_quotient_type_add(i, ideal_quotient_mk_type(i, a), ideal_quotient_mk_type(i, b)).val =
        ideal_quotient_add(i, ideal_quotient_mk_type(i, a).val, ideal_quotient_mk_type(i, b).val)
    ideal_quotient_mk_type_val(i, a)
    ideal_quotient_mk_type_val(i, b)
    ideal_quotient_add(i, ideal_quotient_mk_type(i, a).val, ideal_quotient_mk_type(i, b).val) =
        ideal_quotient_add(i, ideal_quotient_project(i, a), ideal_quotient_project(i, b))
    ideal_quotient_add_projection(i, a, b)
    ideal_quotient_add(i, ideal_quotient_project(i, a), ideal_quotient_project(i, b)) =
        quotient_over_mk(ideal_quotient_relation(i), a + b)
    ideal_quotient_type_add(i, ideal_quotient_mk_type(i, a), ideal_quotient_mk_type(i, b)).val =
        quotient_over_mk(ideal_quotient_relation(i), a + b)
    ideal_quotient_mk_type_val(i, a + b)
    ideal_quotient_mk_type(i, a + b).val = ideal_quotient_project(i, a + b)
    ideal_quotient_project_eq_mk(i, a + b)
    ideal_quotient_project(i, a + b) = quotient_over_mk(ideal_quotient_relation(i), a + b)
    ideal_quotient_mk_type(i, a + b).val = quotient_over_mk(ideal_quotient_relation(i), a + b)
    ideal_quotient_type_ext(i,
        ideal_quotient_type_add(i, ideal_quotient_mk_type(i, a), ideal_quotient_mk_type(i, b)),
        ideal_quotient_mk_type(i, a + b))
}

/// Wrapped multiplication agrees with multiplication of representatives.
theorem ideal_quotient_type_mul_mk[R: CommRing](i: Ideal[R], a: R, b: R) {
    ideal_quotient_type_mul(i, ideal_quotient_mk_type(i, a), ideal_quotient_mk_type(i, b)) =
        ideal_quotient_mk_type(i, a * b)
} by {
    ideal_quotient_type_mul_val(i, ideal_quotient_mk_type(i, a), ideal_quotient_mk_type(i, b))
    ideal_quotient_type_mul(i, ideal_quotient_mk_type(i, a), ideal_quotient_mk_type(i, b)).val =
        ideal_quotient_mul(i, ideal_quotient_mk_type(i, a).val, ideal_quotient_mk_type(i, b).val)
    ideal_quotient_mk_type_val(i, a)
    ideal_quotient_mk_type_val(i, b)
    ideal_quotient_mul(i, ideal_quotient_mk_type(i, a).val, ideal_quotient_mk_type(i, b).val) =
        ideal_quotient_mul(i, ideal_quotient_project(i, a), ideal_quotient_project(i, b))
    ideal_quotient_mul_projection(i, a, b)
    ideal_quotient_mul(i, ideal_quotient_project(i, a), ideal_quotient_project(i, b)) =
        quotient_over_mk(ideal_quotient_relation(i), a * b)
    ideal_quotient_type_mul(i, ideal_quotient_mk_type(i, a), ideal_quotient_mk_type(i, b)).val =
        quotient_over_mk(ideal_quotient_relation(i), a * b)
    ideal_quotient_mk_type_val(i, a * b)
    ideal_quotient_mk_type(i, a * b).val = ideal_quotient_project(i, a * b)
    ideal_quotient_project_eq_mk(i, a * b)
    ideal_quotient_project(i, a * b) = quotient_over_mk(ideal_quotient_relation(i), a * b)
    ideal_quotient_mk_type(i, a * b).val = quotient_over_mk(ideal_quotient_relation(i), a * b)
    ideal_quotient_type_ext(i,
        ideal_quotient_type_mul(i, ideal_quotient_mk_type(i, a), ideal_quotient_mk_type(i, b)),
        ideal_quotient_mk_type(i, a * b))
}

/// Wrapped negation agrees with negation of representatives.
theorem ideal_quotient_type_neg_mk[R: CommRing](i: Ideal[R], a: R) {
    ideal_quotient_type_neg(i, ideal_quotient_mk_type(i, a)) =
        ideal_quotient_mk_type(i, -a)
} by {
    ideal_quotient_type_neg_val(i, ideal_quotient_mk_type(i, a))
    ideal_quotient_type_neg(i, ideal_quotient_mk_type(i, a)).val =
        ideal_quotient_neg(i, ideal_quotient_mk_type(i, a).val)
    ideal_quotient_mk_type_val(i, a)
    ideal_quotient_neg(i, ideal_quotient_mk_type(i, a).val) =
        ideal_quotient_neg(i, ideal_quotient_project(i, a))
    ideal_quotient_neg_projection(i, a)
    ideal_quotient_neg(i, ideal_quotient_project(i, a)) =
        quotient_over_mk(ideal_quotient_relation(i), -a)
    ideal_quotient_type_neg(i, ideal_quotient_mk_type(i, a)).val =
        quotient_over_mk(ideal_quotient_relation(i), -a)
    ideal_quotient_mk_type_val(i, -a)
    ideal_quotient_mk_type(i, -a).val = ideal_quotient_project(i, -a)
    ideal_quotient_project_eq_mk(i, -a)
    ideal_quotient_project(i, -a) = quotient_over_mk(ideal_quotient_relation(i), -a)
    ideal_quotient_mk_type(i, -a).val = quotient_over_mk(ideal_quotient_relation(i), -a)
    ideal_quotient_type_ext(i,
        ideal_quotient_type_neg(i, ideal_quotient_mk_type(i, a)),
        ideal_quotient_mk_type(i, -a))
}

/// The wrapped zero is the projection of the ring zero.
theorem ideal_quotient_type_zero_mk[R: CommRing](i: Ideal[R]) {
    ideal_quotient_type_zero(i) = ideal_quotient_mk_type(i, R.0)
} by {
    ideal_quotient_type_zero_val(i)
    ideal_quotient_type_zero(i).val = ideal_quotient_zero(i)
    ideal_quotient_zero_projection(i)
    ideal_quotient_zero(i) = quotient_over_mk(ideal_quotient_relation(i), R.0)
    ideal_quotient_mk_type_val(i, R.0)
    ideal_quotient_mk_type(i, R.0).val = ideal_quotient_project(i, R.0)
    ideal_quotient_project_eq_mk(i, R.0)
    ideal_quotient_project(i, R.0) = quotient_over_mk(ideal_quotient_relation(i), R.0)
    ideal_quotient_mk_type(i, R.0).val = quotient_over_mk(ideal_quotient_relation(i), R.0)
    ideal_quotient_type_zero(i).val = ideal_quotient_mk_type(i, R.0).val
    ideal_quotient_type_ext(i, ideal_quotient_type_zero(i), ideal_quotient_mk_type(i, R.0))
}

/// The wrapped one is the projection of the ring one.
theorem ideal_quotient_type_one_mk[R: CommRing](i: Ideal[R]) {
    ideal_quotient_type_one(i) = ideal_quotient_mk_type(i, R.1)
} by {
    ideal_quotient_type_one_val(i)
    ideal_quotient_type_one(i).val = ideal_quotient_one(i)
    ideal_quotient_one_projection(i)
    ideal_quotient_one(i) = quotient_over_mk(ideal_quotient_relation(i), R.1)
    ideal_quotient_mk_type_val(i, R.1)
    ideal_quotient_mk_type(i, R.1).val = ideal_quotient_project(i, R.1)
    ideal_quotient_project_eq_mk(i, R.1)
    ideal_quotient_project(i, R.1) = quotient_over_mk(ideal_quotient_relation(i), R.1)
    ideal_quotient_mk_type(i, R.1).val = quotient_over_mk(ideal_quotient_relation(i), R.1)
    ideal_quotient_type_one(i).val = ideal_quotient_mk_type(i, R.1).val
    ideal_quotient_type_ext(i, ideal_quotient_type_one(i), ideal_quotient_mk_type(i, R.1))
}
/// Addition on the quotient ring is associative.
theorem ideal_quotient_type_add_assoc[R: CommRing](i: Ideal[R], x: IdealQuotient[R, i], y: IdealQuotient[R, i], z: IdealQuotient[R, i]) {
    ideal_quotient_type_add(i, ideal_quotient_type_add(i, x, y), z) =
        ideal_quotient_type_add(i, x, ideal_quotient_type_add(i, y, z))
} by {
    ideal_quotient_mk_type_surjective(i, x)
    let a: R satisfy { ideal_quotient_mk_type(i, a) = x }
    ideal_quotient_mk_type_surjective(i, y)
    let b: R satisfy { ideal_quotient_mk_type(i, b) = y }
    ideal_quotient_mk_type_surjective(i, z)
    let c: R satisfy { ideal_quotient_mk_type(i, c) = z }
    ideal_quotient_type_add_mk(i, a, b)
    ideal_quotient_type_add_mk(i, a + b, c)
    ideal_quotient_type_add_mk(i, b, c)
    ideal_quotient_type_add_mk(i, a, b + c)
    (a + b) + c = a + (b + c)
    ideal_quotient_type_add(i, ideal_quotient_type_add(i, x, y), z) =
        ideal_quotient_type_add(i, ideal_quotient_type_add(i, ideal_quotient_mk_type(i, a), ideal_quotient_mk_type(i, b)), ideal_quotient_mk_type(i, c))
    ideal_quotient_type_add(i, ideal_quotient_type_add(i, ideal_quotient_mk_type(i, a), ideal_quotient_mk_type(i, b)), ideal_quotient_mk_type(i, c)) =
        ideal_quotient_type_add(i, ideal_quotient_mk_type(i, a + b), ideal_quotient_mk_type(i, c))
    ideal_quotient_type_add(i, ideal_quotient_mk_type(i, a + b), ideal_quotient_mk_type(i, c)) =
        ideal_quotient_mk_type(i, (a + b) + c)
    ideal_quotient_type_add(i, x, ideal_quotient_type_add(i, y, z)) =
        ideal_quotient_type_add(i, ideal_quotient_mk_type(i, a), ideal_quotient_type_add(i, ideal_quotient_mk_type(i, b), ideal_quotient_mk_type(i, c)))
    ideal_quotient_type_add(i, ideal_quotient_mk_type(i, a), ideal_quotient_type_add(i, ideal_quotient_mk_type(i, b), ideal_quotient_mk_type(i, c))) =
        ideal_quotient_type_add(i, ideal_quotient_mk_type(i, a), ideal_quotient_mk_type(i, b + c))
    ideal_quotient_type_add(i, ideal_quotient_mk_type(i, a), ideal_quotient_mk_type(i, b + c)) =
        ideal_quotient_mk_type(i, a + (b + c))
    ideal_quotient_mk_type(i, (a + b) + c) = ideal_quotient_mk_type(i, a + (b + c))
}

/// Addition on the quotient ring is commutative.
theorem ideal_quotient_type_add_comm[R: CommRing](i: Ideal[R], x: IdealQuotient[R, i], y: IdealQuotient[R, i]) {
    ideal_quotient_type_add(i, x, y) = ideal_quotient_type_add(i, y, x)
} by {
    ideal_quotient_mk_type_surjective(i, x)
    let a: R satisfy { ideal_quotient_mk_type(i, a) = x }
    ideal_quotient_mk_type_surjective(i, y)
    let b: R satisfy { ideal_quotient_mk_type(i, b) = y }
    ideal_quotient_type_add_mk(i, a, b)
    ideal_quotient_type_add_mk(i, b, a)
    a + b = b + a
    ideal_quotient_type_add(i, x, y) =
        ideal_quotient_type_add(i, ideal_quotient_mk_type(i, a), ideal_quotient_mk_type(i, b))
    ideal_quotient_type_add(i, ideal_quotient_mk_type(i, a), ideal_quotient_mk_type(i, b)) =
        ideal_quotient_mk_type(i, a + b)
    ideal_quotient_type_add(i, y, x) =
        ideal_quotient_type_add(i, ideal_quotient_mk_type(i, b), ideal_quotient_mk_type(i, a))
    ideal_quotient_type_add(i, ideal_quotient_mk_type(i, b), ideal_quotient_mk_type(i, a)) =
        ideal_quotient_mk_type(i, b + a)
    ideal_quotient_mk_type(i, a + b) = ideal_quotient_mk_type(i, b + a)
}

/// The quotient zero is a left identity for quotient addition.
theorem ideal_quotient_type_zero_add[R: CommRing](i: Ideal[R], x: IdealQuotient[R, i]) {
    ideal_quotient_type_add(i, ideal_quotient_type_zero(i), x) = x
} by {
    ideal_quotient_mk_type_surjective(i, x)
    let a: R satisfy { ideal_quotient_mk_type(i, a) = x }
    ideal_quotient_type_zero_mk(i)
    ideal_quotient_type_add_mk(i, R.0, a)
    R.0 + a = a
    ideal_quotient_type_add(i, ideal_quotient_type_zero(i), x) =
        ideal_quotient_type_add(i, ideal_quotient_mk_type(i, R.0), ideal_quotient_mk_type(i, a))
    ideal_quotient_type_add(i, ideal_quotient_mk_type(i, R.0), ideal_quotient_mk_type(i, a)) =
        ideal_quotient_mk_type(i, R.0 + a)
    ideal_quotient_mk_type(i, R.0 + a) = ideal_quotient_mk_type(i, a)
    ideal_quotient_mk_type(i, a) = x
}

/// The quotient zero is a right identity for quotient addition.
theorem ideal_quotient_type_add_zero[R: CommRing](i: Ideal[R], x: IdealQuotient[R, i]) {
    ideal_quotient_type_add(i, x, ideal_quotient_type_zero(i)) = x
} by {
    ideal_quotient_mk_type_surjective(i, x)
    let a: R satisfy { ideal_quotient_mk_type(i, a) = x }
    ideal_quotient_type_zero_mk(i)
    ideal_quotient_type_add_mk(i, a, R.0)
    a + R.0 = a
    ideal_quotient_type_add(i, x, ideal_quotient_type_zero(i)) =
        ideal_quotient_type_add(i, ideal_quotient_mk_type(i, a), ideal_quotient_mk_type(i, R.0))
    ideal_quotient_type_add(i, ideal_quotient_mk_type(i, a), ideal_quotient_mk_type(i, R.0)) =
        ideal_quotient_mk_type(i, a + R.0)
    ideal_quotient_mk_type(i, a + R.0) = ideal_quotient_mk_type(i, a)
    ideal_quotient_mk_type(i, a) = x
}

/// Quotient negation is a left inverse for quotient addition.
theorem ideal_quotient_type_neg_add[R: CommRing](i: Ideal[R], x: IdealQuotient[R, i]) {
    ideal_quotient_type_add(i, ideal_quotient_type_neg(i, x), x) = ideal_quotient_type_zero(i)
} by {
    ideal_quotient_mk_type_surjective(i, x)
    let a: R satisfy { ideal_quotient_mk_type(i, a) = x }
    ideal_quotient_type_neg_mk(i, a)
    ideal_quotient_type_add_mk(i, -a, a)
    -a + a = R.0
    ideal_quotient_type_zero_mk(i)
    ideal_quotient_type_add(i, ideal_quotient_type_neg(i, x), x) =
        ideal_quotient_type_add(i, ideal_quotient_type_neg(i, ideal_quotient_mk_type(i, a)), ideal_quotient_mk_type(i, a))
    ideal_quotient_type_add(i, ideal_quotient_type_neg(i, ideal_quotient_mk_type(i, a)), ideal_quotient_mk_type(i, a)) =
        ideal_quotient_type_add(i, ideal_quotient_mk_type(i, -a), ideal_quotient_mk_type(i, a))
    ideal_quotient_type_add(i, ideal_quotient_mk_type(i, -a), ideal_quotient_mk_type(i, a)) =
        ideal_quotient_mk_type(i, -a + a)
    ideal_quotient_mk_type(i, -a + a) = ideal_quotient_mk_type(i, R.0)
    ideal_quotient_mk_type(i, R.0) = ideal_quotient_type_zero(i)
}

/// Quotient negation is a right inverse for quotient addition.
theorem ideal_quotient_type_add_neg[R: CommRing](i: Ideal[R], x: IdealQuotient[R, i]) {
    ideal_quotient_type_add(i, x, ideal_quotient_type_neg(i, x)) = ideal_quotient_type_zero(i)
} by {
    ideal_quotient_mk_type_surjective(i, x)
    let a: R satisfy { ideal_quotient_mk_type(i, a) = x }
    ideal_quotient_type_neg_mk(i, a)
    ideal_quotient_type_add_mk(i, a, -a)
    a + -a = R.0
    ideal_quotient_type_zero_mk(i)
    ideal_quotient_type_add(i, x, ideal_quotient_type_neg(i, x)) =
        ideal_quotient_type_add(i, ideal_quotient_mk_type(i, a), ideal_quotient_type_neg(i, ideal_quotient_mk_type(i, a)))
    ideal_quotient_type_add(i, ideal_quotient_mk_type(i, a), ideal_quotient_type_neg(i, ideal_quotient_mk_type(i, a))) =
        ideal_quotient_type_add(i, ideal_quotient_mk_type(i, a), ideal_quotient_mk_type(i, -a))
    ideal_quotient_type_add(i, ideal_quotient_mk_type(i, a), ideal_quotient_mk_type(i, -a)) =
        ideal_quotient_mk_type(i, a + -a)
    ideal_quotient_mk_type(i, a + -a) = ideal_quotient_mk_type(i, R.0)
    ideal_quotient_mk_type(i, R.0) = ideal_quotient_type_zero(i)
}

/// Multiplication on the quotient ring is associative.
theorem ideal_quotient_type_mul_assoc[R: CommRing](i: Ideal[R], x: IdealQuotient[R, i], y: IdealQuotient[R, i], z: IdealQuotient[R, i]) {
    ideal_quotient_type_mul(i, ideal_quotient_type_mul(i, x, y), z) =
        ideal_quotient_type_mul(i, x, ideal_quotient_type_mul(i, y, z))
} by {
    ideal_quotient_mk_type_surjective(i, x)
    let a: R satisfy { ideal_quotient_mk_type(i, a) = x }
    ideal_quotient_mk_type_surjective(i, y)
    let b: R satisfy { ideal_quotient_mk_type(i, b) = y }
    ideal_quotient_mk_type_surjective(i, z)
    let c: R satisfy { ideal_quotient_mk_type(i, c) = z }
    ideal_quotient_type_mul_mk(i, a, b)
    ideal_quotient_type_mul_mk(i, a * b, c)
    ideal_quotient_type_mul_mk(i, b, c)
    ideal_quotient_type_mul_mk(i, a, b * c)
    (a * b) * c = a * (b * c)
    ideal_quotient_type_mul(i, ideal_quotient_type_mul(i, x, y), z) =
        ideal_quotient_type_mul(i, ideal_quotient_type_mul(i, ideal_quotient_mk_type(i, a), ideal_quotient_mk_type(i, b)), ideal_quotient_mk_type(i, c))
    ideal_quotient_type_mul(i, ideal_quotient_type_mul(i, ideal_quotient_mk_type(i, a), ideal_quotient_mk_type(i, b)), ideal_quotient_mk_type(i, c)) =
        ideal_quotient_type_mul(i, ideal_quotient_mk_type(i, a * b), ideal_quotient_mk_type(i, c))
    ideal_quotient_type_mul(i, ideal_quotient_mk_type(i, a * b), ideal_quotient_mk_type(i, c)) =
        ideal_quotient_mk_type(i, (a * b) * c)
    ideal_quotient_type_mul(i, x, ideal_quotient_type_mul(i, y, z)) =
        ideal_quotient_type_mul(i, ideal_quotient_mk_type(i, a), ideal_quotient_type_mul(i, ideal_quotient_mk_type(i, b), ideal_quotient_mk_type(i, c)))
    ideal_quotient_type_mul(i, ideal_quotient_mk_type(i, a), ideal_quotient_type_mul(i, ideal_quotient_mk_type(i, b), ideal_quotient_mk_type(i, c))) =
        ideal_quotient_type_mul(i, ideal_quotient_mk_type(i, a), ideal_quotient_mk_type(i, b * c))
    ideal_quotient_type_mul(i, ideal_quotient_mk_type(i, a), ideal_quotient_mk_type(i, b * c)) =
        ideal_quotient_mk_type(i, a * (b * c))
    ideal_quotient_mk_type(i, (a * b) * c) = ideal_quotient_mk_type(i, a * (b * c))
}

/// Multiplication on the quotient ring is commutative.
theorem ideal_quotient_type_mul_comm[R: CommRing](i: Ideal[R], x: IdealQuotient[R, i], y: IdealQuotient[R, i]) {
    ideal_quotient_type_mul(i, x, y) = ideal_quotient_type_mul(i, y, x)
} by {
    ideal_quotient_mk_type_surjective(i, x)
    let a: R satisfy { ideal_quotient_mk_type(i, a) = x }
    ideal_quotient_mk_type_surjective(i, y)
    let b: R satisfy { ideal_quotient_mk_type(i, b) = y }
    ideal_quotient_type_mul_mk(i, a, b)
    ideal_quotient_type_mul_mk(i, b, a)
    a * b = b * a
    ideal_quotient_type_mul(i, x, y) =
        ideal_quotient_type_mul(i, ideal_quotient_mk_type(i, a), ideal_quotient_mk_type(i, b))
    ideal_quotient_type_mul(i, ideal_quotient_mk_type(i, a), ideal_quotient_mk_type(i, b)) =
        ideal_quotient_mk_type(i, a * b)
    ideal_quotient_type_mul(i, y, x) =
        ideal_quotient_type_mul(i, ideal_quotient_mk_type(i, b), ideal_quotient_mk_type(i, a))
    ideal_quotient_type_mul(i, ideal_quotient_mk_type(i, b), ideal_quotient_mk_type(i, a)) =
        ideal_quotient_mk_type(i, b * a)
    ideal_quotient_mk_type(i, a * b) = ideal_quotient_mk_type(i, b * a)
}

/// The quotient one is a left identity for quotient multiplication.
theorem ideal_quotient_type_one_mul[R: CommRing](i: Ideal[R], x: IdealQuotient[R, i]) {
    ideal_quotient_type_mul(i, ideal_quotient_type_one(i), x) = x
} by {
    ideal_quotient_mk_type_surjective(i, x)
    let a: R satisfy { ideal_quotient_mk_type(i, a) = x }
    ideal_quotient_type_one_mk(i)
    ideal_quotient_type_mul_mk(i, R.1, a)
    R.1 * a = a
    ideal_quotient_type_mul(i, ideal_quotient_type_one(i), x) =
        ideal_quotient_type_mul(i, ideal_quotient_mk_type(i, R.1), ideal_quotient_mk_type(i, a))
    ideal_quotient_type_mul(i, ideal_quotient_mk_type(i, R.1), ideal_quotient_mk_type(i, a)) =
        ideal_quotient_mk_type(i, R.1 * a)
    ideal_quotient_mk_type(i, R.1 * a) = ideal_quotient_mk_type(i, a)
    ideal_quotient_mk_type(i, a) = x
}

/// The quotient one is a right identity for quotient multiplication.
theorem ideal_quotient_type_mul_one[R: CommRing](i: Ideal[R], x: IdealQuotient[R, i]) {
    ideal_quotient_type_mul(i, x, ideal_quotient_type_one(i)) = x
} by {
    ideal_quotient_mk_type_surjective(i, x)
    let a: R satisfy { ideal_quotient_mk_type(i, a) = x }
    ideal_quotient_type_one_mk(i)
    ideal_quotient_type_mul_mk(i, a, R.1)
    a * R.1 = a
    ideal_quotient_type_mul(i, x, ideal_quotient_type_one(i)) =
        ideal_quotient_type_mul(i, ideal_quotient_mk_type(i, a), ideal_quotient_mk_type(i, R.1))
    ideal_quotient_type_mul(i, ideal_quotient_mk_type(i, a), ideal_quotient_mk_type(i, R.1)) =
        ideal_quotient_mk_type(i, a * R.1)
    ideal_quotient_mk_type(i, a * R.1) = ideal_quotient_mk_type(i, a)
    ideal_quotient_mk_type(i, a) = x
}

/// Quotient multiplication distributes over quotient addition on the left.
theorem ideal_quotient_type_mul_add[R: CommRing](i: Ideal[R], x: IdealQuotient[R, i], y: IdealQuotient[R, i], z: IdealQuotient[R, i]) {
    ideal_quotient_type_mul(i, x, ideal_quotient_type_add(i, y, z)) =
        ideal_quotient_type_add(i,
            ideal_quotient_type_mul(i, x, y),
            ideal_quotient_type_mul(i, x, z))
} by {
    ideal_quotient_mk_type_surjective(i, x)
    let a: R satisfy { ideal_quotient_mk_type(i, a) = x }
    ideal_quotient_mk_type_surjective(i, y)
    let b: R satisfy { ideal_quotient_mk_type(i, b) = y }
    ideal_quotient_mk_type_surjective(i, z)
    let c: R satisfy { ideal_quotient_mk_type(i, c) = z }
    ideal_quotient_type_add_mk(i, b, c)
    ideal_quotient_type_mul_mk(i, a, b + c)
    ideal_quotient_type_mul_mk(i, a, b)
    ideal_quotient_type_mul_mk(i, a, c)
    ideal_quotient_type_add_mk(i, a * b, a * c)
    a * (b + c) = a * b + a * c
    ideal_quotient_type_mul(i, x, ideal_quotient_type_add(i, y, z)) =
        ideal_quotient_type_mul(i, ideal_quotient_mk_type(i, a), ideal_quotient_type_add(i, ideal_quotient_mk_type(i, b), ideal_quotient_mk_type(i, c)))
    ideal_quotient_type_mul(i, ideal_quotient_mk_type(i, a), ideal_quotient_type_add(i, ideal_quotient_mk_type(i, b), ideal_quotient_mk_type(i, c))) =
        ideal_quotient_type_mul(i, ideal_quotient_mk_type(i, a), ideal_quotient_mk_type(i, b + c))
    ideal_quotient_type_mul(i, ideal_quotient_mk_type(i, a), ideal_quotient_mk_type(i, b + c)) =
        ideal_quotient_mk_type(i, a * (b + c))
    ideal_quotient_type_add(i,
        ideal_quotient_type_mul(i, x, y),
        ideal_quotient_type_mul(i, x, z)) =
        ideal_quotient_type_add(i,
            ideal_quotient_type_mul(i, ideal_quotient_mk_type(i, a), ideal_quotient_mk_type(i, b)),
            ideal_quotient_type_mul(i, ideal_quotient_mk_type(i, a), ideal_quotient_mk_type(i, c)))
    ideal_quotient_type_add(i,
        ideal_quotient_type_mul(i, ideal_quotient_mk_type(i, a), ideal_quotient_mk_type(i, b)),
        ideal_quotient_type_mul(i, ideal_quotient_mk_type(i, a), ideal_quotient_mk_type(i, c))) =
        ideal_quotient_type_add(i, ideal_quotient_mk_type(i, a * b), ideal_quotient_mk_type(i, a * c))
    ideal_quotient_type_add(i, ideal_quotient_mk_type(i, a * b), ideal_quotient_mk_type(i, a * c)) =
        ideal_quotient_mk_type(i, a * b + a * c)
    ideal_quotient_mk_type(i, a * (b + c)) = ideal_quotient_mk_type(i, a * b + a * c)
}

/// Quotient multiplication distributes over quotient addition on the right.
theorem ideal_quotient_type_add_mul[R: CommRing](i: Ideal[R], x: IdealQuotient[R, i], y: IdealQuotient[R, i], z: IdealQuotient[R, i]) {
    ideal_quotient_type_mul(i, ideal_quotient_type_add(i, x, y), z) =
        ideal_quotient_type_add(i,
            ideal_quotient_type_mul(i, x, z),
            ideal_quotient_type_mul(i, y, z))
} by {
    ideal_quotient_mk_type_surjective(i, x)
    let a: R satisfy { ideal_quotient_mk_type(i, a) = x }
    ideal_quotient_mk_type_surjective(i, y)
    let b: R satisfy { ideal_quotient_mk_type(i, b) = y }
    ideal_quotient_mk_type_surjective(i, z)
    let c: R satisfy { ideal_quotient_mk_type(i, c) = z }
    ideal_quotient_type_add_mk(i, a, b)
    ideal_quotient_type_mul_mk(i, a + b, c)
    ideal_quotient_type_mul_mk(i, a, c)
    ideal_quotient_type_mul_mk(i, b, c)
    ideal_quotient_type_add_mk(i, a * c, b * c)
    (a + b) * c = a * c + b * c
    ideal_quotient_type_mul(i, ideal_quotient_type_add(i, x, y), z) =
        ideal_quotient_type_mul(i, ideal_quotient_type_add(i, ideal_quotient_mk_type(i, a), ideal_quotient_mk_type(i, b)), ideal_quotient_mk_type(i, c))
    ideal_quotient_type_mul(i, ideal_quotient_type_add(i, ideal_quotient_mk_type(i, a), ideal_quotient_mk_type(i, b)), ideal_quotient_mk_type(i, c)) =
        ideal_quotient_type_mul(i, ideal_quotient_mk_type(i, a + b), ideal_quotient_mk_type(i, c))
    ideal_quotient_type_mul(i, ideal_quotient_mk_type(i, a + b), ideal_quotient_mk_type(i, c)) =
        ideal_quotient_mk_type(i, (a + b) * c)
    ideal_quotient_type_add(i,
        ideal_quotient_type_mul(i, x, z),
        ideal_quotient_type_mul(i, y, z)) =
        ideal_quotient_type_add(i,
            ideal_quotient_type_mul(i, ideal_quotient_mk_type(i, a), ideal_quotient_mk_type(i, c)),
            ideal_quotient_type_mul(i, ideal_quotient_mk_type(i, b), ideal_quotient_mk_type(i, c)))
    ideal_quotient_type_add(i,
        ideal_quotient_type_mul(i, ideal_quotient_mk_type(i, a), ideal_quotient_mk_type(i, c)),
        ideal_quotient_type_mul(i, ideal_quotient_mk_type(i, b), ideal_quotient_mk_type(i, c))) =
        ideal_quotient_type_add(i, ideal_quotient_mk_type(i, a * c), ideal_quotient_mk_type(i, b * c))
    ideal_quotient_type_add(i, ideal_quotient_mk_type(i, a * c), ideal_quotient_mk_type(i, b * c)) =
        ideal_quotient_mk_type(i, a * c + b * c)
    ideal_quotient_mk_type(i, (a + b) * c) = ideal_quotient_mk_type(i, a * c + b * c)
}

/// Multiplying by the quotient zero on the right gives the quotient zero.
theorem ideal_quotient_type_mul_zero[R: CommRing](i: Ideal[R], x: IdealQuotient[R, i]) {
    ideal_quotient_type_mul(i, x, ideal_quotient_type_zero(i)) = ideal_quotient_type_zero(i)
} by {
    ideal_quotient_mk_type_surjective(i, x)
    let a: R satisfy { ideal_quotient_mk_type(i, a) = x }
    ideal_quotient_type_zero_mk(i)
    ideal_quotient_type_mul_mk(i, a, R.0)
    a * R.0 = R.0
    ideal_quotient_type_mul(i, x, ideal_quotient_type_zero(i)) =
        ideal_quotient_type_mul(i, ideal_quotient_mk_type(i, a), ideal_quotient_mk_type(i, R.0))
    ideal_quotient_type_mul(i, ideal_quotient_mk_type(i, a), ideal_quotient_mk_type(i, R.0)) =
        ideal_quotient_mk_type(i, a * R.0)
    ideal_quotient_mk_type(i, a * R.0) = ideal_quotient_mk_type(i, R.0)
    ideal_quotient_mk_type(i, R.0) = ideal_quotient_type_zero(i)
}

/// Multiplying by the quotient zero on the left gives the quotient zero.
theorem ideal_quotient_type_zero_mul[R: CommRing](i: Ideal[R], x: IdealQuotient[R, i]) {
    ideal_quotient_type_mul(i, ideal_quotient_type_zero(i), x) = ideal_quotient_type_zero(i)
} by {
    ideal_quotient_mk_type_surjective(i, x)
    let a: R satisfy { ideal_quotient_mk_type(i, a) = x }
    ideal_quotient_type_zero_mk(i)
    ideal_quotient_type_mul_mk(i, R.0, a)
    R.0 * a = R.0
    ideal_quotient_type_mul(i, ideal_quotient_type_zero(i), x) =
        ideal_quotient_type_mul(i, ideal_quotient_mk_type(i, R.0), ideal_quotient_mk_type(i, a))
    ideal_quotient_type_mul(i, ideal_quotient_mk_type(i, R.0), ideal_quotient_mk_type(i, a)) =
        ideal_quotient_mk_type(i, R.0 * a)
    ideal_quotient_mk_type(i, R.0 * a) = ideal_quotient_mk_type(i, R.0)
    ideal_quotient_mk_type(i, R.0) = ideal_quotient_type_zero(i)
}
/// Typeclass addition on the quotient ring is associative.
theorem ideal_quotient_type_add_assoc_law[R: CommRing](i: Ideal[R], x: IdealQuotient[R, i], y: IdealQuotient[R, i], z: IdealQuotient[R, i]) {
    Add.add(x, Add.add(y, z)) = Add.add(Add.add(x, y), z)
} by {
    ideal_quotient_typeclass_add_eq(i, x, y)
    ideal_quotient_typeclass_add_eq(i, x, ideal_quotient_type_add(i, y, z))
    ideal_quotient_typeclass_add_eq(i, y, z)
    ideal_quotient_typeclass_add_eq(i, ideal_quotient_type_add(i, x, y), z)
    ideal_quotient_type_add_assoc(i, x, y, z)
}

/// Typeclass addition on the quotient ring is commutative.
theorem ideal_quotient_type_add_comm_law[R: CommRing](i: Ideal[R], x: IdealQuotient[R, i], y: IdealQuotient[R, i]) {
    Add.add(x, y) = Add.add(y, x)
} by {
    ideal_quotient_typeclass_add_eq(i, x, y)
    ideal_quotient_typeclass_add_eq(i, y, x)
    ideal_quotient_type_add_comm(i, x, y)
}

/// The typeclass zero is a left identity for quotient addition.
theorem ideal_quotient_type_add_zero_left_law[R: CommRing](i: Ideal[R], x: IdealQuotient[R, i]) {
    Add.add(Zero.0[IdealQuotient[R, i]], x) = x
} by {
    ideal_quotient_typeclass_zero_eq(i)
    ideal_quotient_typeclass_add_eq(i, ideal_quotient_type_zero(i), x)
    ideal_quotient_type_zero_add(i, x)
}

/// The typeclass zero is a right identity for quotient addition.
theorem ideal_quotient_type_add_zero_right_law[R: CommRing](i: Ideal[R], x: IdealQuotient[R, i]) {
    Add.add(x, Zero.0[IdealQuotient[R, i]]) = x
} by {
    ideal_quotient_typeclass_zero_eq(i)
    ideal_quotient_typeclass_add_eq(i, x, ideal_quotient_type_zero(i))
    ideal_quotient_type_add_zero(i, x)
}

/// The typeclass negation is a right additive inverse on the quotient ring.
theorem ideal_quotient_type_add_neg_law[R: CommRing](i: Ideal[R], x: IdealQuotient[R, i]) {
    Add.add(x, Neg.neg(x)) = Zero.0[IdealQuotient[R, i]]
} by {
    ideal_quotient_typeclass_neg_eq(i, x)
    ideal_quotient_typeclass_zero_eq(i)
    ideal_quotient_typeclass_add_eq(i, x, ideal_quotient_type_neg(i, x))
    ideal_quotient_type_add_neg(i, x)
}

/// Quotient-ring elements form an additive semigroup.
instance IdealQuotient[R: CommRing, i: Ideal[R]]: AddSemigroup

/// Quotient-ring elements form an additive commutative semigroup.
instance IdealQuotient[R: CommRing, i: Ideal[R]]: AddCommSemigroup

/// Quotient-ring elements form an additive monoid.
instance IdealQuotient[R: CommRing, i: Ideal[R]]: AddMonoid

/// Quotient-ring elements form an additive commutative monoid.
instance IdealQuotient[R: CommRing, i: Ideal[R]]: AddCommMonoid

/// Quotient-ring elements form an additive group.
instance IdealQuotient[R: CommRing, i: Ideal[R]]: AddGroup

/// Quotient-ring elements form an additive commutative group.
instance IdealQuotient[R: CommRing, i: Ideal[R]]: AddCommGroup

/// Typeclass multiplication on the quotient ring is associative.
theorem ideal_quotient_type_mul_assoc_law[R: CommRing](i: Ideal[R], x: IdealQuotient[R, i], y: IdealQuotient[R, i], z: IdealQuotient[R, i]) {
    Mul.mul(x, Mul.mul(y, z)) = Mul.mul(Mul.mul(x, y), z)
} by {
    ideal_quotient_typeclass_mul_eq(i, x, y)
    ideal_quotient_typeclass_mul_eq(i, x, ideal_quotient_type_mul(i, y, z))
    ideal_quotient_typeclass_mul_eq(i, y, z)
    ideal_quotient_typeclass_mul_eq(i, ideal_quotient_type_mul(i, x, y), z)
    ideal_quotient_type_mul_assoc(i, x, y, z)
}

/// Typeclass multiplication on the quotient ring is commutative.
theorem ideal_quotient_type_mul_comm_law[R: CommRing](i: Ideal[R], x: IdealQuotient[R, i], y: IdealQuotient[R, i]) {
    Mul.mul(x, y) = Mul.mul(y, x)
} by {
    ideal_quotient_typeclass_mul_eq(i, x, y)
    ideal_quotient_typeclass_mul_eq(i, y, x)
    ideal_quotient_type_mul_comm(i, x, y)
}

/// The typeclass one is a left identity for quotient multiplication.
theorem ideal_quotient_type_mul_one_left_law[R: CommRing](i: Ideal[R], x: IdealQuotient[R, i]) {
    Mul.mul(One.1[IdealQuotient[R, i]], x) = x
} by {
    ideal_quotient_typeclass_one_eq(i)
    ideal_quotient_typeclass_mul_eq(i, ideal_quotient_type_one(i), x)
    ideal_quotient_type_one_mul(i, x)
}

/// The typeclass one is a right identity for quotient multiplication.
theorem ideal_quotient_type_mul_one_right_law[R: CommRing](i: Ideal[R], x: IdealQuotient[R, i]) {
    Mul.mul(x, One.1[IdealQuotient[R, i]]) = x
} by {
    ideal_quotient_typeclass_one_eq(i)
    ideal_quotient_typeclass_mul_eq(i, x, ideal_quotient_type_one(i))
    ideal_quotient_type_mul_one(i, x)
}

/// Quotient-ring elements form a multiplicative semigroup.
instance IdealQuotient[R: CommRing, i: Ideal[R]]: Semigroup

/// Quotient-ring elements form a multiplicative commutative semigroup.
instance IdealQuotient[R: CommRing, i: Ideal[R]]: CommSemigroup

/// Quotient-ring elements form a multiplicative monoid.
instance IdealQuotient[R: CommRing, i: Ideal[R]]: Monoid

/// Quotient-ring elements form a multiplicative commutative monoid.
instance IdealQuotient[R: CommRing, i: Ideal[R]]: CommMonoid

/// Typeclass multiplication distributes over typeclass addition on the left.
theorem ideal_quotient_type_mul_add_law[R: CommRing](i: Ideal[R], x: IdealQuotient[R, i], y: IdealQuotient[R, i], z: IdealQuotient[R, i]) {
    Mul.mul(x, Add.add(y, z)) = Add.add(Mul.mul(x, y), Mul.mul(x, z))
} by {
    ideal_quotient_typeclass_add_eq(i, y, z)
    ideal_quotient_typeclass_mul_eq(i, x, ideal_quotient_type_add(i, y, z))
    Mul.mul(x, Add.add(y, z)) = ideal_quotient_type_mul(i, x, ideal_quotient_type_add(i, y, z))
    ideal_quotient_typeclass_mul_eq(i, x, y)
    ideal_quotient_typeclass_mul_eq(i, x, z)
    ideal_quotient_typeclass_add_eq(i, ideal_quotient_type_mul(i, x, y), ideal_quotient_type_mul(i, x, z))
    Add.add(Mul.mul(x, y), Mul.mul(x, z)) =
        ideal_quotient_type_add(i, ideal_quotient_type_mul(i, x, y), ideal_quotient_type_mul(i, x, z))
    ideal_quotient_type_mul_add(i, x, y, z)
}

/// Typeclass multiplication distributes over typeclass addition on the right.
theorem ideal_quotient_type_add_mul_law[R: CommRing](i: Ideal[R], x: IdealQuotient[R, i], y: IdealQuotient[R, i], z: IdealQuotient[R, i]) {
    Mul.mul(Add.add(x, y), z) = Add.add(Mul.mul(x, z), Mul.mul(y, z))
} by {
    ideal_quotient_typeclass_add_eq(i, x, y)
    ideal_quotient_typeclass_mul_eq(i, ideal_quotient_type_add(i, x, y), z)
    Mul.mul(Add.add(x, y), z) = ideal_quotient_type_mul(i, ideal_quotient_type_add(i, x, y), z)
    ideal_quotient_typeclass_mul_eq(i, x, z)
    ideal_quotient_typeclass_mul_eq(i, y, z)
    ideal_quotient_typeclass_add_eq(i, ideal_quotient_type_mul(i, x, z), ideal_quotient_type_mul(i, y, z))
    Add.add(Mul.mul(x, z), Mul.mul(y, z)) =
        ideal_quotient_type_add(i, ideal_quotient_type_mul(i, x, z), ideal_quotient_type_mul(i, y, z))
    ideal_quotient_type_add_mul(i, x, y, z)
}

/// Multiplying by the typeclass zero on the right gives the typeclass zero.
theorem ideal_quotient_type_mul_zero_right_law[R: CommRing](i: Ideal[R], x: IdealQuotient[R, i]) {
    Mul.mul(x, Zero.0[IdealQuotient[R, i]]) = Zero.0[IdealQuotient[R, i]]
} by {
    ideal_quotient_typeclass_zero_eq(i)
    ideal_quotient_typeclass_mul_eq(i, x, ideal_quotient_type_zero(i))
    Mul.mul(x, Zero.0[IdealQuotient[R, i]]) = ideal_quotient_type_mul(i, x, ideal_quotient_type_zero(i))
    ideal_quotient_type_mul_zero(i, x)
    ideal_quotient_type_mul(i, x, ideal_quotient_type_zero(i)) = ideal_quotient_type_zero(i)
    ideal_quotient_type_mul(i, x, ideal_quotient_type_zero(i)) = Zero.0[IdealQuotient[R, i]]
}

/// Multiplying by the typeclass zero on the left gives the typeclass zero.
theorem ideal_quotient_type_zero_mul_law[R: CommRing](i: Ideal[R], x: IdealQuotient[R, i]) {
    Mul.mul(Zero.0[IdealQuotient[R, i]], x) = Zero.0[IdealQuotient[R, i]]
} by {
    ideal_quotient_typeclass_zero_eq(i)
    ideal_quotient_typeclass_mul_eq(i, ideal_quotient_type_zero(i), x)
    Mul.mul(Zero.0[IdealQuotient[R, i]], x) = ideal_quotient_type_mul(i, ideal_quotient_type_zero(i), x)
    ideal_quotient_type_zero_mul(i, x)
    ideal_quotient_type_mul(i, ideal_quotient_type_zero(i), x) = ideal_quotient_type_zero(i)
    ideal_quotient_type_mul(i, ideal_quotient_type_zero(i), x) = Zero.0[IdealQuotient[R, i]]
}

/// Quotient-ring elements form a semiring.
instance IdealQuotient[R: CommRing, i: Ideal[R]]: Semiring

/// The quotient ring `R/I` is a ring.
instance IdealQuotient[R: CommRing, i: Ideal[R]]: Ring

/// The quotient ring `R/I` is a commutative ring.
instance IdealQuotient[R: CommRing, i: Ideal[R]]: CommRing
/// The canonical projection preserves the ring one.
theorem ideal_quotient_mk_type_one[R: CommRing](i: Ideal[R]) {
    ideal_quotient_mk_type(i, R.1) = One.1[IdealQuotient[R, i]]
} by {
    ideal_quotient_type_one_mk(i)
    ideal_quotient_typeclass_one_eq(i)
}

/// The canonical projection preserves addition.
theorem ideal_quotient_mk_type_add[R: CommRing](i: Ideal[R], a: R, b: R) {
    ideal_quotient_mk_type(i, a + b) = ideal_quotient_mk_type(i, a) + ideal_quotient_mk_type(i, b)
} by {
    ideal_quotient_type_add_mk(i, a, b)
    ideal_quotient_typeclass_add_eq(i, ideal_quotient_mk_type(i, a), ideal_quotient_mk_type(i, b))
    ideal_quotient_mk_type(i, a) + ideal_quotient_mk_type(i, b) =
        ideal_quotient_mk_type(i, a + b)
}

/// The canonical projection preserves multiplication.
theorem ideal_quotient_mk_type_mul[R: CommRing](i: Ideal[R], a: R, b: R) {
    ideal_quotient_mk_type(i, a * b) = ideal_quotient_mk_type(i, a) * ideal_quotient_mk_type(i, b)
} by {
    ideal_quotient_type_mul_mk(i, a, b)
    ideal_quotient_typeclass_mul_eq(i, ideal_quotient_mk_type(i, a), ideal_quotient_mk_type(i, b))
    ideal_quotient_mk_type(i, a) * ideal_quotient_mk_type(i, b) =
        ideal_quotient_mk_type(i, a * b)
}

/// The canonical projection from `R` to `R/I` is a ring homomorphism: it
/// preserves the multiplicative identity, addition, and multiplication.
/// (The library `RingHom` structure cannot be instantiated at the dependent
/// codomain `IdealQuotient[R, i]`: typeclass arguments do not accept the term
/// parameter `i`, so the homomorphism conditions are stated directly.)
theorem ideal_quotient_canonical_projection_is_hom[R: CommRing](i: Ideal[R]) {
    ideal_quotient_mk_type(i, R.1) = One.1[IdealQuotient[R, i]] and
    forall(a: R, b: R) {
        ideal_quotient_mk_type(i, a + b) = ideal_quotient_mk_type(i, a) + ideal_quotient_mk_type(i, b) and
            ideal_quotient_mk_type(i, a * b) = ideal_quotient_mk_type(i, a) * ideal_quotient_mk_type(i, b)
    }
} by {
    ideal_quotient_mk_type_one(i)
    forall(a: R, b: R) {
        ideal_quotient_mk_type_add(i, a, b)
        ideal_quotient_mk_type_mul(i, a, b)
        ideal_quotient_mk_type(i, a + b) = ideal_quotient_mk_type(i, a) + ideal_quotient_mk_type(i, b) and
            ideal_quotient_mk_type(i, a * b) = ideal_quotient_mk_type(i, a) * ideal_quotient_mk_type(i, b)
    }
    ideal_quotient_mk_type(i, R.1) = One.1[IdealQuotient[R, i]] and forall(a: R, b: R) {
        ideal_quotient_mk_type(i, a + b) = ideal_quotient_mk_type(i, a) + ideal_quotient_mk_type(i, b) and
            ideal_quotient_mk_type(i, a * b) = ideal_quotient_mk_type(i, a) * ideal_quotient_mk_type(i, b)
    }
}

/// The ideal quotient relation of the kernel ideal is the kernel relation.
theorem ideal_quotient_rel_eq_kernel_relation[R: CommRing, S: CommRing](f: RingHom[R, S], a: R, b: R) {
    ideal_quotient_rel(ring_hom_kernel(f), a, b) = kernel_relation(f.hom, a, b)
} by {
    ideal_quotient_rel_eq_contains_sub(ring_hom_kernel(f), a, b)
    ring_hom_kernel_contains_eq(f, a - b)
    ideal_quotient_rel(ring_hom_kernel(f), a, b) = (f.hom(a - b) = S.0)
    ring_hom_eq_iff_sub_zero(f, b, a)
    (f.hom(b) = f.hom(a)) = (f.hom(a - b) = S.0)
    ideal_quotient_rel(ring_hom_kernel(f), a, b) = (f.hom(b) = f.hom(a))
    kernel_relation(f.hom, a, b) = (f.hom(a) = f.hom(b))
    ideal_quotient_rel(ring_hom_kernel(f), a, b) = kernel_relation(f.hom, a, b)
}

/// The ideal quotient relation of the kernel ideal is the kernel quotient relation.
theorem ideal_quotient_relation_eq_kernel_quotient_relation[R: CommRing, S: CommRing](f: RingHom[R, S]) {
    ideal_quotient_relation(ring_hom_kernel(f)) = kernel_quotient_relation(f.hom)
} by {
    forall(x: R, y: R) {
        ideal_quotient_rel_eq_kernel_relation(f, x, y)
        ideal_quotient_rel(ring_hom_kernel(f), x, y) = kernel_relation(f.hom, x, y)
    }
    binary_function_extensionality(ideal_quotient_rel(ring_hom_kernel(f)), kernel_relation(f.hom))
    ideal_quotient_relation_rel(ring_hom_kernel(f))
    ideal_quotient_relation(ring_hom_kernel(f)).rel = ideal_quotient_rel(ring_hom_kernel(f))
    kernel_quotient_relation_rel(f.hom)
    kernel_quotient_relation(f.hom).rel = kernel_relation(f.hom)
    ideal_quotient_relation(ring_hom_kernel(f)).rel = kernel_quotient_relation(f.hom).rel
    quotient_relation_ext(ideal_quotient_relation(ring_hom_kernel(f)), kernel_quotient_relation(f.hom))
}

/// The ideal-quotient projection by the kernel ideal agrees with the kernel-quotient projection.
theorem ideal_quotient_project_eq_kernel_quotient_project[R: CommRing, S: CommRing](f: RingHom[R, S], a: R) {
    ideal_quotient_project(ring_hom_kernel(f), a) = ring_hom_kernel_quotient_project(f, a)
} by {
    ideal_quotient_project_eq_mk(ring_hom_kernel(f), a)
    ideal_quotient_project(ring_hom_kernel(f), a) =
        quotient_over_mk(ideal_quotient_relation(ring_hom_kernel(f)), a)
    ring_hom_kernel_quotient_project_eq_mk(f, a)
    ring_hom_kernel_quotient_project(f, a) =
        quotient_over_mk(kernel_quotient_relation(f.hom), a)
    ideal_quotient_relation_eq_kernel_quotient_relation(f)
    ideal_quotient_relation(ring_hom_kernel(f)) = kernel_quotient_relation(f.hom)
    quotient_over_mk_fields(ideal_quotient_relation(ring_hom_kernel(f)), a)
    quotient_over_mk(ideal_quotient_relation(ring_hom_kernel(f)), a).qrel =
        ideal_quotient_relation(ring_hom_kernel(f))
    quotient_over_mk_fields(kernel_quotient_relation(f.hom), a)
    quotient_over_mk(kernel_quotient_relation(f.hom), a).qrel = kernel_quotient_relation(f.hom)
    quotient_over_mk(ideal_quotient_relation(ring_hom_kernel(f)), a).qrel =
        quotient_over_mk(kernel_quotient_relation(f.hom), a).qrel
    quotient_over_mk(ideal_quotient_relation(ring_hom_kernel(f)), a).carrier =
        equivalence_class(ideal_quotient_relation(ring_hom_kernel(f)).rel, a)
    quotient_over_mk(kernel_quotient_relation(f.hom), a).carrier =
        equivalence_class(kernel_quotient_relation(f.hom).rel, a)
    quotient_over_mk(ideal_quotient_relation(ring_hom_kernel(f)), a).carrier =
        quotient_over_mk(kernel_quotient_relation(f.hom), a).carrier
    quotient_over_eq_of_fields_eq(
        quotient_over_mk(ideal_quotient_relation(ring_hom_kernel(f)), a),
        quotient_over_mk(kernel_quotient_relation(f.hom), a))
    ideal_quotient_project(ring_hom_kernel(f), a) =
        quotient_over_mk(kernel_quotient_relation(f.hom), a)
    ideal_quotient_project(ring_hom_kernel(f), a) = ring_hom_kernel_quotient_project(f, a)
}

/// The projection to `R/ker(f)` is surjective.
theorem ideal_quotient_kernel_project_surjective[R: CommRing, S: CommRing](f: RingHom[R, S], q: QuotientOver[R]) {
    q.qrel = ideal_quotient_relation(ring_hom_kernel(f)) implies
    exists(a: R) {
        ideal_quotient_project(ring_hom_kernel(f), a) = q
    }
} by {
    if q.qrel = ideal_quotient_relation(ring_hom_kernel(f)) {
        quotient_over_mk_surjective(q)
        let a: R satisfy {
            quotient_over_mk(q.qrel, a) = q
        }
        quotient_over_mk(ideal_quotient_relation(ring_hom_kernel(f)), a) = q
        ideal_quotient_project_eq_mk(ring_hom_kernel(f), a)
        ideal_quotient_project(ring_hom_kernel(f), a) =
            quotient_over_mk(ideal_quotient_relation(ring_hom_kernel(f)), a)
        ideal_quotient_project(ring_hom_kernel(f), a) = q
        exists(a0: R) {
            ideal_quotient_project(ring_hom_kernel(f), a0) = q
        }
    }
}

/// Two elements have equal images under `f` exactly when their cosets in
/// `R/ker(f)` agree: the well-definedness core of the first isomorphism theorem.
theorem ideal_quotient_kernel_first_iso[R: CommRing, S: CommRing](f: RingHom[R, S], a: R, b: R) {
    (ideal_quotient_project(ring_hom_kernel(f), a) = ideal_quotient_project(ring_hom_kernel(f), b)) =
        (f.hom(a) = f.hom(b))
} by {
    ideal_quotient_project_eq_iff_contains_sub(ring_hom_kernel(f), a, b)
    (ideal_quotient_project(ring_hom_kernel(f), a) = ideal_quotient_project(ring_hom_kernel(f), b)) =
        ring_hom_kernel(f).contains(a - b)
    ring_hom_kernel_contains_eq(f, a - b)
    ring_hom_kernel(f).contains(a - b) = (f.hom(a - b) = S.0)
    (ideal_quotient_project(ring_hom_kernel(f), a) = ideal_quotient_project(ring_hom_kernel(f), b)) =
        (f.hom(a - b) = S.0)
    ring_hom_eq_iff_sub_zero(f, b, a)
    (f.hom(b) = f.hom(a)) = (f.hom(a - b) = S.0)
    (ideal_quotient_project(ring_hom_kernel(f), a) = ideal_quotient_project(ring_hom_kernel(f), b)) =
        (f.hom(b) = f.hom(a))
    f.hom(b) = f.hom(a) = (f.hom(a) = f.hom(b))
}

/// The induced map out of `R/ker(f)` agrees with `f` on canonical cosets.
theorem ideal_quotient_kernel_induced_project[R: CommRing, S: CommRing](f: RingHom[R, S], a: R) {
    ring_hom_kernel_quotient_induced(f, ideal_quotient_project(ring_hom_kernel(f), a)) = f.hom(a)
} by {
    ideal_quotient_project_eq_kernel_quotient_project(f, a)
    ring_hom_kernel_quotient_induced_project(f, a)
}

/// The induced map out of `R/ker(f)` is injective.
theorem ideal_quotient_kernel_induced_injective[R: CommRing, S: CommRing](f: RingHom[R, S], a: R, b: R) {
    ring_hom_kernel_quotient_induced(f, ideal_quotient_project(ring_hom_kernel(f), a)) =
        ring_hom_kernel_quotient_induced(f, ideal_quotient_project(ring_hom_kernel(f), b))
    implies ideal_quotient_project(ring_hom_kernel(f), a) = ideal_quotient_project(ring_hom_kernel(f), b)
} by {
    if ring_hom_kernel_quotient_induced(f, ideal_quotient_project(ring_hom_kernel(f), a)) =
        ring_hom_kernel_quotient_induced(f, ideal_quotient_project(ring_hom_kernel(f), b)) {
        ideal_quotient_project_eq_kernel_quotient_project(f, a)
        ideal_quotient_project_eq_kernel_quotient_project(f, b)
        ring_hom_kernel_quotient_induced_injective(f, a, b)
        ring_hom_kernel_quotient_project(f, a) = ring_hom_kernel_quotient_project(f, b)
        ideal_quotient_project(ring_hom_kernel(f), a) = ideal_quotient_project(ring_hom_kernel(f), b)
    }
}

/// Every image value of `f` is reached by the induced map on a coset of `R/ker(f)`.
theorem ideal_quotient_kernel_induced_surjective_onto_image[R: CommRing, S: CommRing](f: RingHom[R, S], y: S) {
    (exists(a: R) { f.hom(a) = y }) implies
    exists(q: QuotientOver[R]) {
        q.qrel = ideal_quotient_relation(ring_hom_kernel(f)) and
        ring_hom_kernel_quotient_induced(f, q) = y
    }
} by {
    if exists(a: R) { f.hom(a) = y } {
        let a: R satisfy { f.hom(a) = y }
        ideal_quotient_kernel_induced_project(f, a)
        ring_hom_kernel_quotient_induced(f, ideal_quotient_project(ring_hom_kernel(f), a)) = f.hom(a)
        ring_hom_kernel_quotient_induced(f, ideal_quotient_project(ring_hom_kernel(f), a)) = y
        ideal_quotient_project_qrel(ring_hom_kernel(f), a)
        ideal_quotient_project(ring_hom_kernel(f), a).qrel = ideal_quotient_relation(ring_hom_kernel(f))
        exists(q: QuotientOver[R]) {
            q.qrel = ideal_quotient_relation(ring_hom_kernel(f)) and
            ring_hom_kernel_quotient_induced(f, q) = y
        }
    }
}

/// The first isomorphism theorem: the induced map identifies two cosets of
/// `R/ker(f)` exactly when the corresponding images under `f` agree, so
/// `R/ker(f)` is isomorphic to the image of `f`.
theorem ideal_quotient_kernel_first_iso_induced[R: CommRing, S: CommRing](f: RingHom[R, S], a: R, b: R) {
    (ring_hom_kernel_quotient_induced(f, ideal_quotient_project(ring_hom_kernel(f), a)) =
        ring_hom_kernel_quotient_induced(f, ideal_quotient_project(ring_hom_kernel(f), b))) =
        (f.hom(a) = f.hom(b))
} by {
    ideal_quotient_kernel_induced_project(f, a)
    ideal_quotient_kernel_induced_project(f, b)
    if ring_hom_kernel_quotient_induced(f, ideal_quotient_project(ring_hom_kernel(f), a)) =
        ring_hom_kernel_quotient_induced(f, ideal_quotient_project(ring_hom_kernel(f), b)) {
        f.hom(a) = f.hom(b)
    }
    if f.hom(a) = f.hom(b) {
        ring_hom_kernel_quotient_induced(f, ideal_quotient_project(ring_hom_kernel(f), a)) =
            ring_hom_kernel_quotient_induced(f, ideal_quotient_project(ring_hom_kernel(f), b))
    }
    (ring_hom_kernel_quotient_induced(f, ideal_quotient_project(ring_hom_kernel(f), a)) =
        ring_hom_kernel_quotient_induced(f, ideal_quotient_project(ring_hom_kernel(f), b))) =
        (f.hom(a) = f.hom(b))
}
