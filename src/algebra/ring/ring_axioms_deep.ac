// Ring-axiom deepening: the standard identities that follow directly from
// the ring axioms — behaviour of negation and subtraction, additive
// cancellation, distributivity consequences such as FOIL, and power laws —
// stated for a general (not necessarily commutative) ring where possible.

from algebra.ring.ring import Ring, mul_zero_left, mul_zero_right, mul_neg_left, mul_neg_right,
    mul_neg_neg, mul_sub_left, mul_sub_right
from algebra.add_group import inverse_inverse, inverse_add, inverse_left, left_cancel, right_cancel
from algebra.add_comm_group import AddCommGroup, sub_add_cancel, sub_eq_zero_imp_eq
from nat import Nat, pow_one, pow_add, pow_zero, add_comm, from_nat
numerals Nat

/// The additive identity is its own additive inverse.
theorem ring_neg_zero[R: Ring] {
    -R.0 = R.0
} by {
    R.0 + -R.0 = R.0
    R.0 + R.0 = R.0
    -R.0 = R.0
}

/// Subtracting an element from itself gives zero.
theorem ring_sub_self[R: Ring](a: R) {
    a - a = R.0
} by {
    a - a = a + -a
    a + -a = R.0
}

/// Subtracting zero leaves an element unchanged.
theorem ring_sub_zero[R: Ring](a: R) {
    a - R.0 = a
} by {
    a - R.0 = a + -R.0
    ring_neg_zero[R]
    -R.0 = R.0
    a + R.0 = a
}

/// Subtracting an element from zero gives its additive inverse.
theorem ring_zero_sub[R: Ring](a: R) {
    R.0 - a = -a
} by {
    R.0 - a = R.0 + -a
    R.0 + -a = -a
}

/// A difference is zero exactly when the two elements are equal.
theorem ring_sub_eq_zero_iff_eq[R: Ring](a: R, b: R) {
    (a - b = R.0) = (a = b)
} by {
    if a - b = R.0 {
        sub_eq_zero_imp_eq(a, b)
        a = b
    }
    if a = b {
        a - b = a - a
        ring_sub_self(a)
        a - a = R.0
        a - b = R.0
    }
    (a - b = R.0) = (a = b)
}

/// The additive inverse of a difference is the reversed difference.
theorem ring_neg_sub[R: Ring](a: R, b: R) {
    -(a - b) = b - a
} by {
    a - b = a + -b
    -(a - b) = -(a + -b)
    inverse_add(a, -b)
    -(a + -b) = --b + -a
    inverse_inverse(b)
    --b = b
    --b + -a = b + -a
    b - a = b + -a
    -(a - b) = b - a
}

/// Subtracting a negated element is adding the element.
theorem ring_sub_neg[R: Ring](a: R, b: R) {
    a - -b = a + b
} by {
    a - -b = a + --b
    inverse_inverse(b)
    --b = b
    a + --b = a + b
}

/// A difference equals the negation of the reversed difference.
theorem ring_sub_eq_neg_sub[R: Ring](a: R, b: R) {
    a - b = -(b - a)
} by {
    ring_neg_sub(b, a)
    -(b - a) = a - b
}

/// An element is zero exactly when its additive inverse is zero.
theorem ring_neg_eq_zero_iff[R: Ring](a: R) {
    (-a = R.0) = (a = R.0)
} by {
    if -a = R.0 {
        inverse_inverse(a)
        --a = a
        -a = R.0
        a = R.0
    }
    if a = R.0 {
        -a = -R.0
        ring_neg_zero[R]
        -R.0 = R.0
        -a = R.0
    }
    (-a = R.0) = (a = R.0)
}

/// Additive inverses are equal exactly when the elements are equal.
theorem ring_neg_eq_neg_iff[R: Ring](a: R, b: R) {
    (-a = -b) = (a = b)
} by {
    if -a = -b {
        inverse_inverse(a)
        --a = a
        inverse_inverse(b)
        --b = b
        -a = -b
        a = b
    }
    if a = b {
        -a = -b
    }
    (-a = -b) = (a = b)
}

/// Association of addition and subtraction: (a + b) - c = a + (b - c).
theorem ring_add_sub_assoc[R: Ring](a: R, b: R, c: R) {
    (a + b) - c = a + (b - c)
} by {
    (a + b) - c = (a + b) + -c
    (a + b) + -c = a + (b + -c)
    b - c = b + -c
    a + (b + -c) = a + (b - c)
    (a + b) - c = a + (b - c)
}

/// Adding back the subtracted element to a difference recovers the first element.
theorem ring_sub_add_cancel[R: Ring](a: R, b: R) {
    (a - b) + b = a
} by {
    sub_add_cancel(a, b)
}

/// Subtracting a summand from a sum recovers the other summand.
theorem ring_add_sub_cancel[R: Ring](a: R, b: R) {
    a + b - b = a
} by {
    a + b - b = a + (b - b)
    ring_sub_self(b)
    b - b = R.0
    a + R.0 = a
    a + b - b = a
}

/// A double subtraction: a - (b - c) = a - b + c.
theorem ring_sub_sub[R: Ring](a: R, b: R, c: R) {
    a - (b - c) = a - b + c
} by {
    b - c = b + -c
    a - (b - c) = a - (b + -c)
    a - (b + -c) = a + -(b + -c)
    inverse_add(b, -c)
    -(b + -c) = --c + -b
    inverse_inverse(c)
    --c = c
    --c + -b = c + -b
    a + (c + -b) = a + c + -b
    a - b + c = a + -b + c
    a + c + -b = a + -b + c
    a - (b - c) = a - b + c
}

/// A sum is zero exactly when the second summand is the negation of the first.
theorem ring_add_eq_zero_iff_eq_neg[R: Ring](a: R, b: R) {
    (a + b = R.0) = (b = -a)
} by {
    if a + b = R.0 {
        inverse_inverse(a)
        left_cancel(a, b, -a)
        a + -a = R.0
        b = -a
    }
    if b = -a {
        a + b = a + -a
        a + -a = R.0
        a + b = R.0
    }
    (a + b = R.0) = (b = -a)
}

/// A sum is zero exactly when the first summand is the negation of the second.
theorem ring_add_eq_zero_iff_eq_neg_left[R: Ring](a: R, b: R) {
    (a + b = R.0) = (a = -b)
} by {
    if a + b = R.0 {
        right_cancel(b, a, -b)
        b + -b = R.0
        a = -b
    }
    if a = -b {
        a + b = -b + b
        inverse_left(b)
        -b + b = R.0
        a + b = R.0
    }
    (a + b = R.0) = (a = -b)
}

/// The negation of a negated sum is the sum.
theorem ring_neg_add_neg_neg[R: Ring](a: R, b: R) {
    -(-a + -b) = a + b
} by {
    inverse_add(-a, -b)
    -(-a + -b) = --b + --a
    inverse_inverse(b)
    --b = b
    inverse_inverse(a)
    --a = a
    --b + --a = b + a
    b + a = a + b
    -(-a + -b) = a + b
}

/// Distributivity recovery: adding a * c back to a * (b - c) gives a * b.
theorem ring_mul_sub_cancel_right[R: Ring](a: R, b: R, c: R) {
    a * (b - c) + a * c = a * b
} by {
    mul_sub_left(a, b, c)
    a * (b - c) = a * b - a * c
    a * (b - c) + a * c = (a * b - a * c) + a * c
    ring_sub_add_cancel(a * b, a * c)
    (a * b - a * c) + a * c = a * b
    a * (b - c) + a * c = a * b
}

/// Distributivity recovery on the right: adding b * c back to (a - b) * c gives a * c.
theorem ring_sub_mul_cancel_right[R: Ring](a: R, b: R, c: R) {
    (a - b) * c + b * c = a * c
} by {
    mul_sub_right(a, b, c)
    (a - b) * c = a * c - b * c
    (a - b) * c + b * c = (a * c - b * c) + b * c
    ring_sub_add_cancel(a * c, b * c)
    (a * c - b * c) + b * c = a * c
    (a - b) * c + b * c = a * c
}

/// A difference with a common subtrahend gives equal elements.
theorem ring_sub_eq_of_sub_eq[R: Ring](a: R, b: R, c: R) {
    a - b = c - b implies a = c
} by {
    if a - b = c - b {
        right_cancel(b, a, c)
        a + -b = c + -b
        a = c
    }
}

/// A difference with a common minuend is zero exactly when the subtrahends are equal.
theorem ring_sub_eq_iff_sub[R: Ring](a: R, b: R, c: R) {
    (a - b = a - c) = (b = c)
} by {
    if a - b = a - c {
        left_cancel(a, b, c)
        a + -b = a + -c
        b = c
    }
    if b = c {
        a - b = a - c
    }
    (a - b = a - c) = (b = c)
}

/// The square of an element is the product with itself.
theorem ring_pow_two[R: Ring](a: R) {
    a.pow(2) = a * a
} by {
    pow_add(a, Nat.1, Nat.1)
    a.pow(Nat.1) * a.pow(Nat.1) = a.pow(2)
    pow_one(a)
    a.pow(Nat.1) = a
    a * a = a.pow(2)
}

/// The cube of an element is the product with itself twice.
theorem ring_pow_three[R: Ring](a: R) {
    a.pow(3) = a * a * a
} by {
    pow_add(a, Nat.2, Nat.1)
    a.pow(Nat.2) * a.pow(Nat.1) = a.pow(3)
    ring_pow_two(a)
    a.pow(2) = a * a
    pow_one(a)
    a.pow(Nat.1) = a
    a * a * a = a.pow(3)
}

/// The square of a negated element is the square of the element.
theorem ring_neg_pow_two[R: Ring](a: R) {
    (-a).pow(2) = a.pow(2)
} by {
    ring_pow_two(-a)
    (-a).pow(2) = -a * -a
    mul_neg_neg(a, a)
    -a * -a = a * a
    ring_pow_two(a)
    a.pow(2) = a * a
    (-a).pow(2) = a.pow(2)
}

/// The square of negative one is one.
theorem ring_neg_one_pow_two[R: Ring] {
    (-R.1).pow(2) = R.1
} by {
    ring_neg_pow_two(R.1)
    (-R.1).pow(2) = R.1.pow(2)
    ring_pow_two(R.1)
    R.1.pow(2) = R.1 * R.1
    R.1 * R.1 = R.1
    (-R.1).pow(2) = R.1
}

/// FOIL: (a + b) * (c + d) = a*c + a*d + b*c + b*d.
theorem ring_foil[R: Ring](a: R, b: R, c: R, d: R) {
    (a + b) * (c + d) = a * c + a * d + b * c + b * d
} by {
    (a + b) * (c + d) = (a + b) * c + (a + b) * d
    (a + b) * c = a * c + b * c
    (a + b) * d = a * d + b * d
    (a + b) * (c + d) = (a * c + b * c) + (a * d + b * d)
    (a * c + b * c) + (a * d + b * d) = a * c + (b * c + (a * d + b * d))
    b * c + (a * d + b * d) = b * c + a * d + b * d
    b * c + a * d = a * d + b * c
    a * c + (a * d + (b * c + b * d)) = a * c + a * d + b * c + b * d
    (a * c + b * c) + (a * d + b * d) = a * c + a * d + b * c + b * d
    (a + b) * (c + d) = a * c + a * d + b * c + b * d
}

/// Powers of the same element commute: a^m * a^n = a^n * a^m.
theorem ring_pow_mul_comm[R: Ring](a: R, m: Nat, n: Nat) {
    a.pow(m) * a.pow(n) = a.pow(n) * a.pow(m)
} by {
    pow_add(a, m, n)
    a.pow(m) * a.pow(n) = a.pow(m + n)
    pow_add(a, n, m)
    a.pow(n) * a.pow(m) = a.pow(n + m)
    add_comm(m, n)
    m + n = n + m
    a.pow(m + n) = a.pow(n + m)
    a.pow(m) * a.pow(n) = a.pow(n) * a.pow(m)
}

/// Multiplying a difference by a zero difference on the left gives zero.
theorem ring_sub_self_mul_left[R: Ring](a: R, b: R) {
    (a - a) * b = R.0
} by {
    ring_sub_self(a)
    a - a = R.0
    (a - a) * b = R.0 * b
    mul_zero_left(b)
    R.0 * b = R.0
    (a - a) * b = R.0
}

/// Multiplying a zero difference on the right gives zero.
theorem ring_mul_sub_self_right[R: Ring](a: R, b: R) {
    a * (b - b) = R.0
} by {
    ring_sub_self(b)
    b - b = R.0
    a * (b - b) = a * R.0
    mul_zero_right(a)
    a * R.0 = R.0
    a * (b - b) = R.0
}

/// The second power of a two-fold sum: (a + a).pow(2) = a.pow(2) + a.pow(2) + a.pow(2) + a.pow(2).
theorem ring_double_square[R: Ring](a: R) {
    (a + a).pow(2) = a.pow(2) + a.pow(2) + a.pow(2) + a.pow(2)
} by {
    ring_pow_two(a + a)
    (a + a).pow(2) = (a + a) * (a + a)
    ring_foil(a, a, a, a)
    (a + a) * (a + a) = a * a + a * a + a * a + a * a
    ring_pow_two(a)
    a.pow(2) = a * a
    a * a + a * a + a * a + a * a = a.pow(2) + a.pow(2) + a.pow(2) + a.pow(2)
    (a + a).pow(2) = a.pow(2) + a.pow(2) + a.pow(2) + a.pow(2)
}

/// Doubling distributes over addition: 2 * (a + b) = 2 * a + 2 * b.
theorem ring_two_mul_add[R: Ring](a: R, b: R) {
    (R.1 + R.1) * (a + b) = (R.1 + R.1) * a + (R.1 + R.1) * b
} by {
    (R.1 + R.1) * (a + b) = (R.1 + R.1) * a + (R.1 + R.1) * b
}

/// The image of two under the natural embedding is one plus one.
theorem ring_from_nat_two[R: Ring] {
    from_nat[R](2) = R.1 + R.1
} by {
    from_nat[R](Nat.0.suc.suc) = from_nat[R](Nat.0.suc) + R.1
    from_nat[R](Nat.0.suc) = from_nat[R](Nat.0) + R.1
    from_nat[R](Nat.0) = R.0
    from_nat[R](2) = R.1 + R.1
}

/// Doubling by the embedded two agrees with adding an element to itself.
theorem ring_two_mul_from_nat[R: Ring](a: R) {
    from_nat[R](2) * a = a + a
} by {
    ring_from_nat_two[R]
    from_nat[R](2) = R.1 + R.1
    (R.1 + R.1) * a = R.1 * a + R.1 * a
    R.1 * a = a
    (R.1 + R.1) * a = a + a
    from_nat[R](2) * a = a + a
}
