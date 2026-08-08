/// Quotient ring operations: addition, negation, multiplication, zero, and one
/// on `QuotientOver[R]` for a commutative ring `R` modulo an ideal `I`.

from comm_ring import CommRing
from nat import Nat, pow_zero
from algebra.ring.ideal import Ideal, ideal_contains_neg, principal_ideal,
    bundled_principal_ideal_contains_eq
from data.basic.equivalence import QuotientRelation, QuotientOver, quotient_over_mk,
    quotient_relation_new_round_trip,
    quotient_unary_respects, quotient_binary_respects,
    quotient_unary_respects_eq_respects_unary_op,
    quotient_binary_respects_eq_respects_binary_op,
    quotient_over_binary_op, quotient_over_unary_op, quotient_over_const,
    quotient_over_binary_op_projection, quotient_over_binary_op_projection_left,
    quotient_over_binary_op_projection_right,
    quotient_over_unary_op_projection, quotient_over_unary_op_projection_compatible,
    quotient_over_const_projection, quotient_over_mk_eq_of_rel, rel_of_quotient_over_mk_eq,
    quotient_over_mk_eq_iff_rel
from data.basic.relation_basic import is_equivalence
from data.basic.relation_transport import respects_unary_op, respects_binary_op,
    respects_add_eq_respects_binary_op, respects_mul_eq_respects_binary_op
from data.basic.quotient_algebra import ideal_quotient_rel, ideal_quotient_rel_eq_contains_sub,
    ideal_quotient_rel_is_equivalence, ideal_quotient_rel_respects_add,
    ideal_quotient_rel_respects_mul, ideal_quotient_rel_add_step,
    ideal_quotient_neg_sub_eq_swap

/// The ideal quotient relation, packaged as a `QuotientRelation[R]`.
let ideal_quotient_relation[R: CommRing](i: Ideal[R]) -> result: QuotientRelation[R] satisfy {
    QuotientRelation[R].new(ideal_quotient_rel(i)) = Option.some(result)
} by {
    ideal_quotient_rel_is_equivalence(i)
}

/// The underlying relation of `ideal_quotient_relation` is `ideal_quotient_rel`.
theorem ideal_quotient_relation_rel[R: CommRing](i: Ideal[R]) {
    ideal_quotient_relation(i).rel = ideal_quotient_rel(i)
} by {
    quotient_relation_new_round_trip(ideal_quotient_rel(i), ideal_quotient_relation(i))
}

/// The ideal quotient relation is compatible with negation.
theorem ideal_quotient_rel_respects_neg[R: CommRing](i: Ideal[R]) {
    respects_unary_op(ideal_quotient_rel(i), R.neg)
} by {
    forall(a: R, b: R) {
        if ideal_quotient_rel(i, a, b) {
            ideal_quotient_rel_eq_contains_sub(i, a, b)
            ideal_contains_neg(i, a - b)
            ideal_quotient_rel_eq_contains_sub(i, -a, -b)
            ideal_quotient_rel(i, R.neg(a), R.neg(b))
        }
    }
}

/// The ideal quotient relation respects addition as a `QuotientRelation`.
theorem ideal_quotient_relation_respects_add[R: CommRing](i: Ideal[R]) {
    quotient_binary_respects(ideal_quotient_relation(i), R.add)
} by {
    quotient_binary_respects_eq_respects_binary_op(ideal_quotient_relation(i), R.add)
    ideal_quotient_rel_respects_add(i)
    respects_add_eq_respects_binary_op(ideal_quotient_rel(i))
    ideal_quotient_relation_rel(i)
}

/// The ideal quotient relation respects multiplication as a `QuotientRelation`.
theorem ideal_quotient_relation_respects_mul[R: CommRing](i: Ideal[R]) {
    quotient_binary_respects(ideal_quotient_relation(i), R.mul)
} by {
    quotient_binary_respects_eq_respects_binary_op(ideal_quotient_relation(i), R.mul)
    ideal_quotient_rel_respects_mul(i)
    respects_mul_eq_respects_binary_op(ideal_quotient_rel(i))
    ideal_quotient_relation_rel(i)
}

/// The ideal quotient relation respects negation as a `QuotientRelation`.
theorem ideal_quotient_relation_respects_neg[R: CommRing](i: Ideal[R]) {
    quotient_unary_respects(ideal_quotient_relation(i), R.neg)
} by {
    quotient_unary_respects_eq_respects_unary_op(ideal_quotient_relation(i), R.neg)
    ideal_quotient_rel_respects_neg(i)
    ideal_quotient_relation_rel(i)
}

/// The ideal quotient relation is compatible with subtraction.
theorem ideal_quotient_rel_respects_sub[R: CommRing](i: Ideal[R]) {
    respects_binary_op(ideal_quotient_rel(i), R.sub)
} by {
    forall(a1: R, b1: R, a2: R, b2: R) {
        if ideal_quotient_rel(i, a1, b1) and ideal_quotient_rel(i, a2, b2) {
            ideal_quotient_rel_respects_neg(i)
            ideal_quotient_rel(i, -a2, -b2)
            ideal_quotient_rel_add_step(i, a1, b1, -a2, -b2)
            ideal_quotient_rel(i, R.sub(a1, a2), R.sub(b1, b2))
        }
    }
}

/// The ideal quotient relation respects subtraction as a `QuotientRelation`.
theorem ideal_quotient_relation_respects_sub[R: CommRing](i: Ideal[R]) {
    quotient_binary_respects(ideal_quotient_relation(i), R.sub)
} by {
    quotient_binary_respects_eq_respects_binary_op(ideal_quotient_relation(i), R.sub)
    ideal_quotient_rel_respects_sub(i)
    ideal_quotient_relation_rel(i)
}

/// Addition on the quotient ring `R/I`.
define ideal_quotient_add[R: CommRing](i: Ideal[R]) ->
    (QuotientOver[R], QuotientOver[R]) -> QuotientOver[R] {
    quotient_over_binary_op(R.add)
}

/// Quotient addition agrees with addition of representatives.
theorem ideal_quotient_add_projection[R: CommRing](i: Ideal[R], a: R, b: R) {
    ideal_quotient_add(i,
        quotient_over_mk(ideal_quotient_relation(i), a),
        quotient_over_mk(ideal_quotient_relation(i), b)
    ) = quotient_over_mk(ideal_quotient_relation(i), a + b)
} by {
    ideal_quotient_relation_respects_add(i)
    quotient_over_binary_op_projection(ideal_quotient_relation(i), R.add, a, b)
    R.add(a, b) = a + b
}

/// Quotient addition respects related left representatives.
theorem ideal_quotient_add_projection_left[R: CommRing](i: Ideal[R], a1: R, a2: R, b: R) {
    ideal_quotient_rel(i, a1, a2) implies
    ideal_quotient_add(i,
        quotient_over_mk(ideal_quotient_relation(i), a1),
        quotient_over_mk(ideal_quotient_relation(i), b)
    ) = ideal_quotient_add(i,
        quotient_over_mk(ideal_quotient_relation(i), a2),
        quotient_over_mk(ideal_quotient_relation(i), b)
    )
} by {
    if ideal_quotient_rel(i, a1, a2) {
        ideal_quotient_relation_rel(i)
        ideal_quotient_relation(i).rel(a1, a2)
        ideal_quotient_relation_respects_add(i)
        quotient_over_binary_op_projection_left(ideal_quotient_relation(i), R.add, a1, a2, b)
    }
}

/// Quotient addition respects related right representatives.
theorem ideal_quotient_add_projection_right[R: CommRing](i: Ideal[R], a: R, b1: R, b2: R) {
    ideal_quotient_rel(i, b1, b2) implies
    ideal_quotient_add(i,
        quotient_over_mk(ideal_quotient_relation(i), a),
        quotient_over_mk(ideal_quotient_relation(i), b1)
    ) = ideal_quotient_add(i,
        quotient_over_mk(ideal_quotient_relation(i), a),
        quotient_over_mk(ideal_quotient_relation(i), b2)
    )
} by {
    if ideal_quotient_rel(i, b1, b2) {
        ideal_quotient_relation_rel(i)
        ideal_quotient_relation(i).rel(b1, b2)
        ideal_quotient_relation_respects_add(i)
        quotient_over_binary_op_projection_right(ideal_quotient_relation(i), R.add, a, b1, b2)
    }
}

/// Multiplication on the quotient ring `R/I`.
define ideal_quotient_mul[R: CommRing](i: Ideal[R]) ->
    (QuotientOver[R], QuotientOver[R]) -> QuotientOver[R] {
    quotient_over_binary_op(R.mul)
}

/// Quotient multiplication agrees with multiplication of representatives.
theorem ideal_quotient_mul_projection[R: CommRing](i: Ideal[R], a: R, b: R) {
    ideal_quotient_mul(i,
        quotient_over_mk(ideal_quotient_relation(i), a),
        quotient_over_mk(ideal_quotient_relation(i), b)
    ) = quotient_over_mk(ideal_quotient_relation(i), a * b)
} by {
    ideal_quotient_relation_respects_mul(i)
    quotient_over_binary_op_projection(ideal_quotient_relation(i), R.mul, a, b)
    R.mul(a, b) = a * b
}

/// Quotient multiplication respects related left representatives.
theorem ideal_quotient_mul_projection_left[R: CommRing](i: Ideal[R], a1: R, a2: R, b: R) {
    ideal_quotient_rel(i, a1, a2) implies
    ideal_quotient_mul(i,
        quotient_over_mk(ideal_quotient_relation(i), a1),
        quotient_over_mk(ideal_quotient_relation(i), b)
    ) = ideal_quotient_mul(i,
        quotient_over_mk(ideal_quotient_relation(i), a2),
        quotient_over_mk(ideal_quotient_relation(i), b)
    )
} by {
    if ideal_quotient_rel(i, a1, a2) {
        ideal_quotient_relation_rel(i)
        ideal_quotient_relation(i).rel(a1, a2)
        ideal_quotient_relation_respects_mul(i)
        quotient_over_binary_op_projection_left(ideal_quotient_relation(i), R.mul, a1, a2, b)
    }
}

/// Quotient multiplication respects related right representatives.
theorem ideal_quotient_mul_projection_right[R: CommRing](i: Ideal[R], a: R, b1: R, b2: R) {
    ideal_quotient_rel(i, b1, b2) implies
    ideal_quotient_mul(i,
        quotient_over_mk(ideal_quotient_relation(i), a),
        quotient_over_mk(ideal_quotient_relation(i), b1)
    ) = ideal_quotient_mul(i,
        quotient_over_mk(ideal_quotient_relation(i), a),
        quotient_over_mk(ideal_quotient_relation(i), b2)
    )
} by {
    if ideal_quotient_rel(i, b1, b2) {
        ideal_quotient_relation_rel(i)
        ideal_quotient_relation(i).rel(b1, b2)
        ideal_quotient_relation_respects_mul(i)
        quotient_over_binary_op_projection_right(ideal_quotient_relation(i), R.mul, a, b1, b2)
    }
}

/// Negation on the quotient ring `R/I`.
define ideal_quotient_neg[R: CommRing](i: Ideal[R]) ->
    QuotientOver[R] -> QuotientOver[R] {
    quotient_over_unary_op(ideal_quotient_relation(i), R.neg)
}

/// Quotient negation agrees with negation of representatives.
theorem ideal_quotient_neg_projection[R: CommRing](i: Ideal[R], a: R) {
    ideal_quotient_neg(i, quotient_over_mk(ideal_quotient_relation(i), a)) =
        quotient_over_mk(ideal_quotient_relation(i), -a)
} by {
    ideal_quotient_relation_respects_neg(i)
    quotient_over_unary_op_projection(ideal_quotient_relation(i), R.neg, a)
    R.neg(a) = -a
}

/// Quotient negation respects related representatives.
theorem ideal_quotient_neg_projection_compatible[R: CommRing](i: Ideal[R], a: R, b: R) {
    ideal_quotient_rel(i, a, b) implies
    ideal_quotient_neg(i, quotient_over_mk(ideal_quotient_relation(i), a)) =
        ideal_quotient_neg(i, quotient_over_mk(ideal_quotient_relation(i), b))
} by {
    if ideal_quotient_rel(i, a, b) {
        ideal_quotient_relation_rel(i)
        ideal_quotient_relation(i).rel(a, b)
        ideal_quotient_relation_respects_neg(i)
        quotient_over_unary_op_projection_compatible(ideal_quotient_relation(i), R.neg, a, b)
    }
}

/// Subtraction on the quotient ring `R/I`.
define ideal_quotient_sub[R: CommRing](i: Ideal[R]) ->
    (QuotientOver[R], QuotientOver[R]) -> QuotientOver[R] {
    quotient_over_binary_op(R.sub)
}

/// Quotient subtraction agrees with subtraction of representatives.
theorem ideal_quotient_sub_projection[R: CommRing](i: Ideal[R], a: R, b: R) {
    ideal_quotient_sub(i,
        quotient_over_mk(ideal_quotient_relation(i), a),
        quotient_over_mk(ideal_quotient_relation(i), b)
    ) = quotient_over_mk(ideal_quotient_relation(i), a - b)
} by {
    ideal_quotient_relation_respects_sub(i)
    quotient_over_binary_op_projection(ideal_quotient_relation(i), R.sub, a, b)
}

/// Quotient subtraction respects related left representatives.
theorem ideal_quotient_sub_projection_left[R: CommRing](i: Ideal[R], a1: R, a2: R, b: R) {
    ideal_quotient_rel(i, a1, a2) implies
    ideal_quotient_sub(i,
        quotient_over_mk(ideal_quotient_relation(i), a1),
        quotient_over_mk(ideal_quotient_relation(i), b)
    ) = ideal_quotient_sub(i,
        quotient_over_mk(ideal_quotient_relation(i), a2),
        quotient_over_mk(ideal_quotient_relation(i), b)
    )
} by {
    if ideal_quotient_rel(i, a1, a2) {
        ideal_quotient_relation_rel(i)
        ideal_quotient_relation(i).rel(a1, a2)
        ideal_quotient_relation_respects_sub(i)
        quotient_over_binary_op_projection_left(ideal_quotient_relation(i), R.sub, a1, a2, b)
    }
}

/// Quotient subtraction respects related right representatives.
theorem ideal_quotient_sub_projection_right[R: CommRing](i: Ideal[R], a: R, b1: R, b2: R) {
    ideal_quotient_rel(i, b1, b2) implies
    ideal_quotient_sub(i,
        quotient_over_mk(ideal_quotient_relation(i), a),
        quotient_over_mk(ideal_quotient_relation(i), b1)
    ) = ideal_quotient_sub(i,
        quotient_over_mk(ideal_quotient_relation(i), a),
        quotient_over_mk(ideal_quotient_relation(i), b2)
    )
} by {
    if ideal_quotient_rel(i, b1, b2) {
        ideal_quotient_relation_rel(i)
        ideal_quotient_relation(i).rel(b1, b2)
        ideal_quotient_relation_respects_sub(i)
        quotient_over_binary_op_projection_right(ideal_quotient_relation(i), R.sub, a, b1, b2)
    }
}

/// The zero element of the quotient ring `R/I`.
define ideal_quotient_zero[R: CommRing](i: Ideal[R]) -> QuotientOver[R] {
    quotient_over_mk(ideal_quotient_relation(i), R.0)
}

/// The one element of the quotient ring `R/I`.
define ideal_quotient_one[R: CommRing](i: Ideal[R]) -> QuotientOver[R] {
    quotient_over_mk(ideal_quotient_relation(i), R.1)
}

/// The quotient zero is the projection of the ring zero.
theorem ideal_quotient_zero_projection[R: CommRing](i: Ideal[R]) {
    ideal_quotient_zero(i) = quotient_over_mk(ideal_quotient_relation(i), R.0)
}

/// The quotient one is the projection of the ring one.
theorem ideal_quotient_one_projection[R: CommRing](i: Ideal[R]) {
    ideal_quotient_one(i) = quotient_over_mk(ideal_quotient_relation(i), R.1)
}

/// The canonical projection from a ring to its quotient by an ideal.
define ideal_quotient_project[R: CommRing](i: Ideal[R], a: R) -> QuotientOver[R] {
    quotient_over_mk(ideal_quotient_relation(i), a)
}

/// The canonical projection is the underlying quotient projection.
theorem ideal_quotient_project_eq_mk[R: CommRing](i: Ideal[R], a: R) {
    ideal_quotient_project(i, a) = quotient_over_mk(ideal_quotient_relation(i), a)
}

/// Related representatives have the same canonical projection.
theorem ideal_quotient_project_eq_of_rel[R: CommRing](i: Ideal[R], a: R, b: R) {
    ideal_quotient_rel(i, a, b) implies ideal_quotient_project(i, a) = ideal_quotient_project(i, b)
} by {
    if ideal_quotient_rel(i, a, b) {
        ideal_quotient_relation_rel(i)
        ideal_quotient_relation(i).rel(a, b)
        quotient_over_mk_eq_of_rel(ideal_quotient_relation(i), a, b)
        ideal_quotient_project(i, a) = ideal_quotient_project(i, b)
    }
}

/// Equality of canonical projections gives the ideal quotient relation.
theorem ideal_quotient_rel_of_project_eq[R: CommRing](i: Ideal[R], a: R, b: R) {
    ideal_quotient_project(i, a) = ideal_quotient_project(i, b) implies ideal_quotient_rel(i, a, b)
} by {
    if ideal_quotient_project(i, a) = ideal_quotient_project(i, b) {
        quotient_over_mk(ideal_quotient_relation(i), a) = quotient_over_mk(ideal_quotient_relation(i), b)
        rel_of_quotient_over_mk_eq(ideal_quotient_relation(i), a, b)
        ideal_quotient_relation_rel(i)
        ideal_quotient_rel(i, a, b)
    }
}

/// Equality of canonical projections is exactly the ideal quotient relation.
theorem ideal_quotient_project_eq_iff_rel[R: CommRing](i: Ideal[R], a: R, b: R) {
    (ideal_quotient_project(i, a) = ideal_quotient_project(i, b)) = ideal_quotient_rel(i, a, b)
} by {
    if ideal_quotient_project(i, a) = ideal_quotient_project(i, b) {
        ideal_quotient_rel_of_project_eq(i, a, b)
        ideal_quotient_rel(i, a, b)
    }
    if ideal_quotient_rel(i, a, b) {
        ideal_quotient_project_eq_of_rel(i, a, b)
        ideal_quotient_project(i, a) = ideal_quotient_project(i, b)
    }
    quotient_over_mk_eq_iff_rel(ideal_quotient_relation(i), a, b)
    ideal_quotient_relation_rel(i)
}

/// Equality of canonical projections is membership of the difference in the ideal.
theorem ideal_quotient_project_eq_iff_contains_sub[R: CommRing](i: Ideal[R], a: R, b: R) {
    (ideal_quotient_project(i, a) = ideal_quotient_project(i, b)) = i.contains(a - b)
} by {
    ideal_quotient_project_eq_iff_rel(i, a, b)
    ideal_quotient_rel_eq_contains_sub(i, a, b)
}

/// Ideal membership of a representative difference gives equality of projections.
theorem ideal_quotient_project_eq_of_contains_sub[R: CommRing](i: Ideal[R], a: R, b: R) {
    i.contains(a - b) implies ideal_quotient_project(i, a) = ideal_quotient_project(i, b)
} by {
    if i.contains(a - b) {
        ideal_quotient_project_eq_iff_contains_sub(i, a, b)
    }
}

/// Equality of projections gives ideal membership of the representative difference.
theorem ideal_quotient_contains_sub_of_project_eq[R: CommRing](i: Ideal[R], a: R, b: R) {
    ideal_quotient_project(i, a) = ideal_quotient_project(i, b) implies i.contains(a - b)
} by {
    if ideal_quotient_project(i, a) = ideal_quotient_project(i, b) {
        ideal_quotient_project_eq_iff_contains_sub(i, a, b)
    }
}

/// Equality modulo a principal ideal is the existence of a multiple of the generator.
theorem ideal_quotient_project_eq_iff_principal_multiple[R: CommRing](m: R, a: R, b: R) {
    (ideal_quotient_project(Ideal[R].principal(m), a) =
        ideal_quotient_project(Ideal[R].principal(m), b)) =
        exists(q: R) { a - b = q * m }
} by {
    ideal_quotient_project_eq_iff_contains_sub(Ideal[R].principal(m), a, b)
    bundled_principal_ideal_contains_eq(m, a - b)
}

/// A principal-ideal multiple of the difference gives equality in the quotient.
theorem ideal_quotient_project_eq_of_principal_multiple[R: CommRing](m: R, a: R, b: R) {
    exists(q: R) { a - b = q * m } implies
        ideal_quotient_project(Ideal[R].principal(m), a) =
        ideal_quotient_project(Ideal[R].principal(m), b)
} by {
    if exists(q: R) { a - b = q * m } {
        ideal_quotient_project_eq_iff_principal_multiple(m, a, b)
    }
}

/// Equality in a principal-ideal quotient gives a multiple of the difference.
theorem principal_multiple_of_ideal_quotient_project_eq[R: CommRing](m: R, a: R, b: R) {
    ideal_quotient_project(Ideal[R].principal(m), a) =
        ideal_quotient_project(Ideal[R].principal(m), b)
        implies exists(q: R) { a - b = q * m }
} by {
    if ideal_quotient_project(Ideal[R].principal(m), a) =
        ideal_quotient_project(Ideal[R].principal(m), b) {
        ideal_quotient_project_eq_iff_principal_multiple(m, a, b)
    }
}

/// Relation to zero is membership in the ideal.
theorem ideal_quotient_rel_zero_right[R: CommRing](i: Ideal[R], a: R) {
    ideal_quotient_rel(i, a, R.0) = i.contains(a)
} by {
    ideal_quotient_rel_eq_contains_sub(i, a, R.0)
    a - R.0 = a
}

/// Relation from zero is membership in the ideal.
theorem ideal_quotient_rel_zero_left[R: CommRing](i: Ideal[R], a: R) {
    ideal_quotient_rel(i, R.0, a) = i.contains(a)
} by {
    ideal_quotient_rel_eq_contains_sub(i, R.0, a)
    if ideal_quotient_rel(i, R.0, a) {
        i.contains(-a)
        ideal_contains_neg(i, -a)
        -(-a) = a
        i.contains(a)
    }
    if i.contains(a) {
        ideal_contains_neg(i, a)
        ideal_quotient_rel(i, R.0, a)
    }
}

/// A representative projects to quotient zero exactly when it lies in the ideal.
theorem ideal_quotient_project_eq_zero_iff_contains[R: CommRing](i: Ideal[R], a: R) {
    (ideal_quotient_project(i, a) = ideal_quotient_zero(i)) = i.contains(a)
} by {
    ideal_quotient_zero_projection(i)
    ideal_quotient_project_eq_iff_rel(i, a, R.0)
    ideal_quotient_rel_zero_right(i, a)
}

/// Quotient zero equals a representative projection exactly when the representative lies in the ideal.
theorem ideal_quotient_zero_eq_project_iff_contains[R: CommRing](i: Ideal[R], a: R) {
    (ideal_quotient_zero(i) = ideal_quotient_project(i, a)) = i.contains(a)
} by {
    ideal_quotient_zero_projection(i)
    ideal_quotient_project_eq_iff_rel(i, R.0, a)
    ideal_quotient_rel_zero_left(i, a)
}

/// Elements of the ideal project to quotient zero.
theorem ideal_quotient_project_eq_zero_of_contains[R: CommRing](i: Ideal[R], a: R) {
    i.contains(a) implies ideal_quotient_project(i, a) = ideal_quotient_zero(i)
} by {
    if i.contains(a) {
        ideal_quotient_project_eq_zero_iff_contains(i, a)
    }
}

/// A representative whose projection is quotient zero lies in the ideal.
theorem ideal_quotient_contains_of_project_eq_zero[R: CommRing](i: Ideal[R], a: R) {
    ideal_quotient_project(i, a) = ideal_quotient_zero(i) implies i.contains(a)
} by {
    if ideal_quotient_project(i, a) = ideal_quotient_zero(i) {
        ideal_quotient_project_eq_zero_iff_contains(i, a)
    }
}

/// Elements of the ideal have projection equal to quotient zero.
theorem ideal_quotient_zero_eq_project_of_contains[R: CommRing](i: Ideal[R], a: R) {
    i.contains(a) implies ideal_quotient_zero(i) = ideal_quotient_project(i, a)
} by {
    if i.contains(a) {
        ideal_quotient_zero_eq_project_iff_contains(i, a)
    }
}

/// A representative equal to quotient zero by projection lies in the ideal.
theorem ideal_quotient_contains_of_zero_eq_project[R: CommRing](i: Ideal[R], a: R) {
    ideal_quotient_zero(i) = ideal_quotient_project(i, a) implies i.contains(a)
} by {
    if ideal_quotient_zero(i) = ideal_quotient_project(i, a) {
        ideal_quotient_zero_eq_project_iff_contains(i, a)
    }
}

/// The zero fiber of a principal-ideal quotient is the set of multiples of the generator.
theorem ideal_quotient_project_eq_zero_iff_principal_multiple[R: CommRing](m: R, a: R) {
    (ideal_quotient_project(Ideal[R].principal(m), a) =
        ideal_quotient_zero(Ideal[R].principal(m))) =
        exists(q: R) { a = q * m }
} by {
    ideal_quotient_project_eq_zero_iff_contains(Ideal[R].principal(m), a)
    bundled_principal_ideal_contains_eq(m, a)
}

/// A multiple of the principal-ideal generator projects to quotient zero.
theorem ideal_quotient_project_eq_zero_of_principal_multiple[R: CommRing](m: R, a: R) {
    exists(q: R) { a = q * m } implies
        ideal_quotient_project(Ideal[R].principal(m), a) =
        ideal_quotient_zero(Ideal[R].principal(m))
} by {
    if exists(q: R) { a = q * m } {
        ideal_quotient_project_eq_zero_iff_principal_multiple(m, a)
    }
}

/// A representative that projects to zero modulo a principal ideal is a multiple of the generator.
theorem principal_multiple_of_ideal_quotient_project_eq_zero[R: CommRing](m: R, a: R) {
    ideal_quotient_project(Ideal[R].principal(m), a) =
        ideal_quotient_zero(Ideal[R].principal(m))
        implies exists(q: R) { a = q * m }
} by {
    if ideal_quotient_project(Ideal[R].principal(m), a) =
        ideal_quotient_zero(Ideal[R].principal(m)) {
        ideal_quotient_project_eq_zero_iff_principal_multiple(m, a)
    }
}

/// The projected difference is quotient zero exactly when the two representatives are related.
theorem ideal_quotient_project_sub_eq_zero_iff_rel[R: CommRing](i: Ideal[R], a: R, b: R) {
    (ideal_quotient_project(i, a - b) = ideal_quotient_zero(i)) = ideal_quotient_rel(i, a, b)
} by {
    ideal_quotient_project_eq_zero_iff_contains(i, a - b)
    ideal_quotient_rel_eq_contains_sub(i, a, b)
}

/// The projected difference is quotient zero exactly when the difference lies in the ideal.
theorem ideal_quotient_project_sub_eq_zero_iff_contains_sub[R: CommRing](i: Ideal[R], a: R, b: R) {
    (ideal_quotient_project(i, a - b) = ideal_quotient_zero(i)) = i.contains(a - b)
} by {
    ideal_quotient_project_eq_zero_iff_contains(i, a - b)
}

/// Related representatives have projected difference equal to quotient zero.
theorem ideal_quotient_project_sub_eq_zero_of_rel[R: CommRing](i: Ideal[R], a: R, b: R) {
    ideal_quotient_rel(i, a, b) implies
    ideal_quotient_project(i, a - b) = ideal_quotient_zero(i)
} by {
    if ideal_quotient_rel(i, a, b) {
        ideal_quotient_project_sub_eq_zero_iff_rel(i, a, b)
    }
}

/// A projected difference equal to quotient zero gives the ideal quotient relation.
theorem ideal_quotient_rel_of_project_sub_eq_zero[R: CommRing](i: Ideal[R], a: R, b: R) {
    ideal_quotient_project(i, a - b) = ideal_quotient_zero(i) implies
    ideal_quotient_rel(i, a, b)
} by {
    if ideal_quotient_project(i, a - b) = ideal_quotient_zero(i) {
        ideal_quotient_project_sub_eq_zero_iff_rel(i, a, b)
    }
}

/// The projected difference is quotient zero exactly when the two projections are equal.
theorem ideal_quotient_project_sub_eq_zero_iff_project_eq[R: CommRing](i: Ideal[R], a: R, b: R) {
    (ideal_quotient_project(i, a - b) = ideal_quotient_zero(i)) =
        (ideal_quotient_project(i, a) = ideal_quotient_project(i, b))
} by {
    ideal_quotient_project_sub_eq_zero_iff_rel(i, a, b)
    ideal_quotient_project_eq_iff_rel(i, a, b)
}

/// Equal projections have projected difference equal to quotient zero.
theorem ideal_quotient_project_sub_eq_zero_of_project_eq[R: CommRing](i: Ideal[R], a: R, b: R) {
    ideal_quotient_project(i, a) = ideal_quotient_project(i, b) implies
    ideal_quotient_project(i, a - b) = ideal_quotient_zero(i)
} by {
    if ideal_quotient_project(i, a) = ideal_quotient_project(i, b) {
        ideal_quotient_project_sub_eq_zero_iff_project_eq(i, a, b)
    }
}

/// A projected difference equal to quotient zero gives equality of projections.
theorem ideal_quotient_project_eq_of_project_sub_eq_zero[R: CommRing](i: Ideal[R], a: R, b: R) {
    ideal_quotient_project(i, a - b) = ideal_quotient_zero(i) implies
    ideal_quotient_project(i, a) = ideal_quotient_project(i, b)
} by {
    if ideal_quotient_project(i, a - b) = ideal_quotient_zero(i) {
        ideal_quotient_project_sub_eq_zero_iff_project_eq(i, a, b)
    }
}

/// The canonical projection sends zero to quotient zero.
theorem ideal_quotient_project_zero[R: CommRing](i: Ideal[R]) {
    ideal_quotient_project(i, R.0) = ideal_quotient_zero(i)
} by {
    ideal_quotient_zero_projection(i)
}

/// The canonical projection sends one to quotient one.
theorem ideal_quotient_project_one[R: CommRing](i: Ideal[R]) {
    ideal_quotient_project(i, R.1) = ideal_quotient_one(i)
} by {
    ideal_quotient_one_projection(i)
}

/// The canonical projection preserves addition.
theorem ideal_quotient_project_add[R: CommRing](i: Ideal[R], a: R, b: R) {
    ideal_quotient_project(i, a + b) =
    ideal_quotient_add(i, ideal_quotient_project(i, a), ideal_quotient_project(i, b))
} by {
    ideal_quotient_add_projection(i, a, b)
}

/// The canonical projection preserves multiplication.
theorem ideal_quotient_project_mul[R: CommRing](i: Ideal[R], a: R, b: R) {
    ideal_quotient_project(i, a * b) =
    ideal_quotient_mul(i, ideal_quotient_project(i, a), ideal_quotient_project(i, b))
} by {
    ideal_quotient_mul_projection(i, a, b)
}

/// The canonical projection preserves negation.
theorem ideal_quotient_project_neg[R: CommRing](i: Ideal[R], a: R) {
    ideal_quotient_project(i, -a) =
    ideal_quotient_neg(i, ideal_quotient_project(i, a))
} by {
    ideal_quotient_neg_projection(i, a)
}

/// The canonical projection preserves subtraction.
theorem ideal_quotient_project_sub[R: CommRing](i: Ideal[R], a: R, b: R) {
    ideal_quotient_project(i, a - b) =
    ideal_quotient_sub(i, ideal_quotient_project(i, a), ideal_quotient_project(i, b))
} by {
    ideal_quotient_sub_projection(i, a, b)
}

/// Quotient addition of canonical projections respects related left representatives.
theorem ideal_quotient_project_add_left_of_rel[R: CommRing](
    i: Ideal[R], a1: R, a2: R, b: R
) {
    ideal_quotient_rel(i, a1, a2) implies
    ideal_quotient_add(i, ideal_quotient_project(i, a1), ideal_quotient_project(i, b)) =
    ideal_quotient_add(i, ideal_quotient_project(i, a2), ideal_quotient_project(i, b))
} by {
    if ideal_quotient_rel(i, a1, a2) {
        ideal_quotient_add_projection_left(i, a1, a2, b)
    }
}

/// Quotient addition of canonical projections respects related right representatives.
theorem ideal_quotient_project_add_right_of_rel[R: CommRing](
    i: Ideal[R], a: R, b1: R, b2: R
) {
    ideal_quotient_rel(i, b1, b2) implies
    ideal_quotient_add(i, ideal_quotient_project(i, a), ideal_quotient_project(i, b1)) =
    ideal_quotient_add(i, ideal_quotient_project(i, a), ideal_quotient_project(i, b2))
} by {
    if ideal_quotient_rel(i, b1, b2) {
        ideal_quotient_add_projection_right(i, a, b1, b2)
    }
}

/// Quotient multiplication of canonical projections respects related left representatives.
theorem ideal_quotient_project_mul_left_of_rel[R: CommRing](
    i: Ideal[R], a1: R, a2: R, b: R
) {
    ideal_quotient_rel(i, a1, a2) implies
    ideal_quotient_mul(i, ideal_quotient_project(i, a1), ideal_quotient_project(i, b)) =
    ideal_quotient_mul(i, ideal_quotient_project(i, a2), ideal_quotient_project(i, b))
} by {
    if ideal_quotient_rel(i, a1, a2) {
        ideal_quotient_mul_projection_left(i, a1, a2, b)
    }
}

/// Quotient multiplication of canonical projections respects related right representatives.
theorem ideal_quotient_project_mul_right_of_rel[R: CommRing](
    i: Ideal[R], a: R, b1: R, b2: R
) {
    ideal_quotient_rel(i, b1, b2) implies
    ideal_quotient_mul(i, ideal_quotient_project(i, a), ideal_quotient_project(i, b1)) =
    ideal_quotient_mul(i, ideal_quotient_project(i, a), ideal_quotient_project(i, b2))
} by {
    if ideal_quotient_rel(i, b1, b2) {
        ideal_quotient_mul_projection_right(i, a, b1, b2)
    }
}

/// Quotient negation of canonical projections respects related representatives.
theorem ideal_quotient_project_neg_of_rel[R: CommRing](i: Ideal[R], a: R, b: R) {
    ideal_quotient_rel(i, a, b) implies
    ideal_quotient_neg(i, ideal_quotient_project(i, a)) =
    ideal_quotient_neg(i, ideal_quotient_project(i, b))
} by {
    if ideal_quotient_rel(i, a, b) {
        ideal_quotient_neg_projection_compatible(i, a, b)
    }
}

/// Quotient subtraction of canonical projections respects related left representatives.
theorem ideal_quotient_project_sub_left_of_rel[R: CommRing](
    i: Ideal[R], a1: R, a2: R, b: R
) {
    ideal_quotient_rel(i, a1, a2) implies
    ideal_quotient_sub(i, ideal_quotient_project(i, a1), ideal_quotient_project(i, b)) =
    ideal_quotient_sub(i, ideal_quotient_project(i, a2), ideal_quotient_project(i, b))
} by {
    if ideal_quotient_rel(i, a1, a2) {
        ideal_quotient_sub_projection_left(i, a1, a2, b)
    }
}

/// Quotient subtraction of canonical projections respects related right representatives.
theorem ideal_quotient_project_sub_right_of_rel[R: CommRing](
    i: Ideal[R], a: R, b1: R, b2: R
) {
    ideal_quotient_rel(i, b1, b2) implies
    ideal_quotient_sub(i, ideal_quotient_project(i, a), ideal_quotient_project(i, b1)) =
    ideal_quotient_sub(i, ideal_quotient_project(i, a), ideal_quotient_project(i, b2))
} by {
    if ideal_quotient_rel(i, b1, b2) {
        ideal_quotient_sub_projection_right(i, a, b1, b2)
    }
}

/// Subtracting quotient zero on the right fixes a projected representative.
theorem ideal_quotient_sub_zero[R: CommRing](i: Ideal[R], a: R) {
    ideal_quotient_sub(i,
        quotient_over_mk(ideal_quotient_relation(i), a),
        ideal_quotient_zero(i)) =
        quotient_over_mk(ideal_quotient_relation(i), a)
} by {
    ideal_quotient_sub_projection(i, a, R.0)
    -R.0 = R.0
    a - R.0 = a
}

/// Subtracting a projected representative from quotient zero gives the projected negation.
theorem ideal_quotient_zero_sub[R: CommRing](i: Ideal[R], a: R) {
    ideal_quotient_sub(i,
        ideal_quotient_zero(i),
        quotient_over_mk(ideal_quotient_relation(i), a)) =
        quotient_over_mk(ideal_quotient_relation(i), -a)
} by {
    ideal_quotient_sub_projection(i, R.0, a)
}

/// A projected representative subtracted from itself is quotient zero.
theorem ideal_quotient_sub_self[R: CommRing](i: Ideal[R], a: R) {
    ideal_quotient_sub(i,
        quotient_over_mk(ideal_quotient_relation(i), a),
        quotient_over_mk(ideal_quotient_relation(i), a)) =
        ideal_quotient_zero(i)
} by {
    ideal_quotient_sub_projection(i, a, a)
    a - a = R.0
}

/// Quotient subtraction agrees with quotient addition of the quotient negation.
theorem ideal_quotient_sub_eq_add_neg[R: CommRing](i: Ideal[R], a: R, b: R) {
    ideal_quotient_sub(i,
        quotient_over_mk(ideal_quotient_relation(i), a),
        quotient_over_mk(ideal_quotient_relation(i), b)) =
    ideal_quotient_add(i,
        quotient_over_mk(ideal_quotient_relation(i), a),
        ideal_quotient_neg(i, quotient_over_mk(ideal_quotient_relation(i), b)))
} by {
    ideal_quotient_sub_projection(i, a, b)
    ideal_quotient_neg_projection(i, b)
    ideal_quotient_add_projection(i, a, -b)
    quotient_over_mk(ideal_quotient_relation(i), a - b) =
        quotient_over_mk(ideal_quotient_relation(i), a + -b)
}

/// Addition on the ideal quotient is associative.
theorem ideal_quotient_add_assoc[R: CommRing](i: Ideal[R], a: R, b: R, c: R) {
    ideal_quotient_add(i,
        ideal_quotient_add(i,
            quotient_over_mk(ideal_quotient_relation(i), a),
            quotient_over_mk(ideal_quotient_relation(i), b)),
        quotient_over_mk(ideal_quotient_relation(i), c)) =
    ideal_quotient_add(i,
        quotient_over_mk(ideal_quotient_relation(i), a),
        ideal_quotient_add(i,
            quotient_over_mk(ideal_quotient_relation(i), b),
            quotient_over_mk(ideal_quotient_relation(i), c)))
} by {
    ideal_quotient_add_projection(i, a, b)
    ideal_quotient_add_projection(i, a + b, c)
    ideal_quotient_add_projection(i, b, c)
    ideal_quotient_add_projection(i, a, b + c)
    quotient_over_mk(ideal_quotient_relation(i), (a + b) + c) =
        quotient_over_mk(ideal_quotient_relation(i), a + (b + c))
}

/// Addition on the ideal quotient is commutative.
theorem ideal_quotient_add_comm[R: CommRing](i: Ideal[R], a: R, b: R) {
    ideal_quotient_add(i,
        quotient_over_mk(ideal_quotient_relation(i), a),
        quotient_over_mk(ideal_quotient_relation(i), b)) =
    ideal_quotient_add(i,
        quotient_over_mk(ideal_quotient_relation(i), b),
        quotient_over_mk(ideal_quotient_relation(i), a))
} by {
    ideal_quotient_add_projection(i, a, b)
    ideal_quotient_add_projection(i, b, a)
}

/// The quotient zero is a left identity for quotient addition.
theorem ideal_quotient_zero_add[R: CommRing](i: Ideal[R], a: R) {
    ideal_quotient_add(i,
        ideal_quotient_zero(i),
        quotient_over_mk(ideal_quotient_relation(i), a)) =
        quotient_over_mk(ideal_quotient_relation(i), a)
} by {
    ideal_quotient_add_projection(i, R.0, a)
}

/// The quotient zero is a right identity for quotient addition.
theorem ideal_quotient_add_zero[R: CommRing](i: Ideal[R], a: R) {
    ideal_quotient_add(i,
        quotient_over_mk(ideal_quotient_relation(i), a),
        ideal_quotient_zero(i)) =
        quotient_over_mk(ideal_quotient_relation(i), a)
} by {
    ideal_quotient_add_projection(i, a, R.0)
}

/// The quotient negation is a left inverse for quotient addition.
theorem ideal_quotient_neg_add[R: CommRing](i: Ideal[R], a: R) {
    ideal_quotient_add(i,
        ideal_quotient_neg(i, quotient_over_mk(ideal_quotient_relation(i), a)),
        quotient_over_mk(ideal_quotient_relation(i), a)) =
        ideal_quotient_zero(i)
} by {
    ideal_quotient_neg_projection(i, a)
    ideal_quotient_add_projection(i, -a, a)
    quotient_over_mk(ideal_quotient_relation(i), -a + a) =
        quotient_over_mk(ideal_quotient_relation(i), R.0)
}

/// The quotient negation is a right inverse for quotient addition.
theorem ideal_quotient_add_neg[R: CommRing](i: Ideal[R], a: R) {
    ideal_quotient_add(i,
        quotient_over_mk(ideal_quotient_relation(i), a),
        ideal_quotient_neg(i, quotient_over_mk(ideal_quotient_relation(i), a))) =
        ideal_quotient_zero(i)
} by {
    ideal_quotient_neg_projection(i, a)
    ideal_quotient_add_projection(i, a, -a)
    quotient_over_mk(ideal_quotient_relation(i), a + -a) =
        quotient_over_mk(ideal_quotient_relation(i), R.0)
}

/// A quotient negation cancels a matching left summand.
theorem ideal_quotient_neg_add_cancel_left[R: CommRing](i: Ideal[R], a: R, b: R) {
    ideal_quotient_add(i,
        ideal_quotient_neg(i, quotient_over_mk(ideal_quotient_relation(i), a)),
        ideal_quotient_add(i,
            quotient_over_mk(ideal_quotient_relation(i), a),
            quotient_over_mk(ideal_quotient_relation(i), b))) =
        quotient_over_mk(ideal_quotient_relation(i), b)
} by {
    ideal_quotient_neg_projection(i, a)
    ideal_quotient_add_projection(i, a, b)
    ideal_quotient_add_projection(i, -a, a + b)
    -a + (a + b) = b
}

/// A quotient negation cancels a matching right summand.
theorem ideal_quotient_add_neg_cancel_right[R: CommRing](i: Ideal[R], a: R, b: R) {
    ideal_quotient_add(i,
        ideal_quotient_add(i,
            quotient_over_mk(ideal_quotient_relation(i), b),
            quotient_over_mk(ideal_quotient_relation(i), a)),
        ideal_quotient_neg(i, quotient_over_mk(ideal_quotient_relation(i), a))) =
        quotient_over_mk(ideal_quotient_relation(i), b)
} by {
    ideal_quotient_add_projection(i, b, a)
    ideal_quotient_neg_projection(i, a)
    ideal_quotient_add_projection(i, b + a, -a)
    (b + a) + -a = b
}

/// Subtracting the right summand of a quotient sum cancels it.
theorem ideal_quotient_add_sub_cancel_right_summand[R: CommRing](i: Ideal[R], a: R, b: R) {
    ideal_quotient_sub(i,
        ideal_quotient_add(i,
            quotient_over_mk(ideal_quotient_relation(i), a),
            quotient_over_mk(ideal_quotient_relation(i), b)),
        quotient_over_mk(ideal_quotient_relation(i), b)) =
        quotient_over_mk(ideal_quotient_relation(i), a)
} by {
    ideal_quotient_add_projection(i, a, b)
    ideal_quotient_sub_projection(i, a + b, b)
    (a + b) - b = a
}

/// Multiplication on the ideal quotient is associative.
theorem ideal_quotient_mul_assoc[R: CommRing](i: Ideal[R], a: R, b: R, c: R) {
    ideal_quotient_mul(i,
        ideal_quotient_mul(i,
            quotient_over_mk(ideal_quotient_relation(i), a),
            quotient_over_mk(ideal_quotient_relation(i), b)),
        quotient_over_mk(ideal_quotient_relation(i), c)) =
    ideal_quotient_mul(i,
        quotient_over_mk(ideal_quotient_relation(i), a),
        ideal_quotient_mul(i,
            quotient_over_mk(ideal_quotient_relation(i), b),
            quotient_over_mk(ideal_quotient_relation(i), c)))
} by {
    ideal_quotient_mul_projection(i, a, b)
    ideal_quotient_mul_projection(i, a * b, c)
    ideal_quotient_mul_projection(i, b, c)
    ideal_quotient_mul_projection(i, a, b * c)
    quotient_over_mk(ideal_quotient_relation(i), (a * b) * c) =
        quotient_over_mk(ideal_quotient_relation(i), a * (b * c))
}

/// Multiplication on the ideal quotient is commutative.
theorem ideal_quotient_mul_comm[R: CommRing](i: Ideal[R], a: R, b: R) {
    ideal_quotient_mul(i,
        quotient_over_mk(ideal_quotient_relation(i), a),
        quotient_over_mk(ideal_quotient_relation(i), b)) =
    ideal_quotient_mul(i,
        quotient_over_mk(ideal_quotient_relation(i), b),
        quotient_over_mk(ideal_quotient_relation(i), a))
} by {
    ideal_quotient_mul_projection(i, a, b)
    ideal_quotient_mul_projection(i, b, a)
}

/// The quotient one is a left identity for quotient multiplication.
theorem ideal_quotient_one_mul[R: CommRing](i: Ideal[R], a: R) {
    ideal_quotient_mul(i,
        ideal_quotient_one(i),
        quotient_over_mk(ideal_quotient_relation(i), a)) =
        quotient_over_mk(ideal_quotient_relation(i), a)
} by {
    ideal_quotient_mul_projection(i, R.1, a)
}

/// The quotient one is a right identity for quotient multiplication.
theorem ideal_quotient_mul_one[R: CommRing](i: Ideal[R], a: R) {
    ideal_quotient_mul(i,
        quotient_over_mk(ideal_quotient_relation(i), a),
        ideal_quotient_one(i)) =
        quotient_over_mk(ideal_quotient_relation(i), a)
} by {
    ideal_quotient_mul_projection(i, a, R.1)
}

/// Quotient multiplication distributes over quotient addition on the left.
theorem ideal_quotient_mul_add[R: CommRing](i: Ideal[R], a: R, b: R, c: R) {
    ideal_quotient_mul(i,
        quotient_over_mk(ideal_quotient_relation(i), a),
        ideal_quotient_add(i,
            quotient_over_mk(ideal_quotient_relation(i), b),
            quotient_over_mk(ideal_quotient_relation(i), c))) =
    ideal_quotient_add(i,
        ideal_quotient_mul(i,
            quotient_over_mk(ideal_quotient_relation(i), a),
            quotient_over_mk(ideal_quotient_relation(i), b)),
        ideal_quotient_mul(i,
            quotient_over_mk(ideal_quotient_relation(i), a),
            quotient_over_mk(ideal_quotient_relation(i), c)))
} by {
    ideal_quotient_add_projection(i, b, c)
    ideal_quotient_mul_projection(i, a, b + c)
    ideal_quotient_mul_projection(i, a, b)
    ideal_quotient_mul_projection(i, a, c)
    ideal_quotient_add_projection(i, a * b, a * c)
    quotient_over_mk(ideal_quotient_relation(i), a * (b + c)) =
        quotient_over_mk(ideal_quotient_relation(i), a * b + a * c)
}

/// Quotient multiplication distributes over quotient addition on the right.
theorem ideal_quotient_add_mul[R: CommRing](i: Ideal[R], a: R, b: R, c: R) {
    ideal_quotient_mul(i,
        ideal_quotient_add(i,
            quotient_over_mk(ideal_quotient_relation(i), a),
            quotient_over_mk(ideal_quotient_relation(i), b)),
        quotient_over_mk(ideal_quotient_relation(i), c)) =
    ideal_quotient_add(i,
        ideal_quotient_mul(i,
            quotient_over_mk(ideal_quotient_relation(i), a),
            quotient_over_mk(ideal_quotient_relation(i), c)),
        ideal_quotient_mul(i,
            quotient_over_mk(ideal_quotient_relation(i), b),
            quotient_over_mk(ideal_quotient_relation(i), c)))
} by {
    ideal_quotient_add_projection(i, a, b)
    ideal_quotient_mul_projection(i, a + b, c)
    ideal_quotient_mul_projection(i, a, c)
    ideal_quotient_mul_projection(i, b, c)
    ideal_quotient_add_projection(i, a * c, b * c)
    quotient_over_mk(ideal_quotient_relation(i), (a + b) * c) =
        quotient_over_mk(ideal_quotient_relation(i), a * c + b * c)
}

/// Quotient multiplication distributes over quotient subtraction on the left.
theorem ideal_quotient_mul_sub[R: CommRing](i: Ideal[R], a: R, b: R, c: R) {
    ideal_quotient_mul(i,
        quotient_over_mk(ideal_quotient_relation(i), a),
        ideal_quotient_sub(i,
            quotient_over_mk(ideal_quotient_relation(i), b),
            quotient_over_mk(ideal_quotient_relation(i), c))) =
    ideal_quotient_sub(i,
        ideal_quotient_mul(i,
            quotient_over_mk(ideal_quotient_relation(i), a),
            quotient_over_mk(ideal_quotient_relation(i), b)),
        ideal_quotient_mul(i,
            quotient_over_mk(ideal_quotient_relation(i), a),
            quotient_over_mk(ideal_quotient_relation(i), c)))
} by {
    ideal_quotient_sub_projection(i, b, c)
    ideal_quotient_mul_projection(i, a, b - c)
    ideal_quotient_mul_projection(i, a, b)
    ideal_quotient_mul_projection(i, a, c)
    ideal_quotient_sub_projection(i, a * b, a * c)
    quotient_over_mk(ideal_quotient_relation(i), a * (b - c)) =
        quotient_over_mk(ideal_quotient_relation(i), a * b - a * c)
}

/// Quotient multiplication distributes over quotient subtraction on the right.
theorem ideal_quotient_sub_mul[R: CommRing](i: Ideal[R], a: R, b: R, c: R) {
    ideal_quotient_mul(i,
        ideal_quotient_sub(i,
            quotient_over_mk(ideal_quotient_relation(i), a),
            quotient_over_mk(ideal_quotient_relation(i), b)),
        quotient_over_mk(ideal_quotient_relation(i), c)) =
    ideal_quotient_sub(i,
        ideal_quotient_mul(i,
            quotient_over_mk(ideal_quotient_relation(i), a),
            quotient_over_mk(ideal_quotient_relation(i), c)),
        ideal_quotient_mul(i,
            quotient_over_mk(ideal_quotient_relation(i), b),
            quotient_over_mk(ideal_quotient_relation(i), c)))
} by {
    ideal_quotient_sub_projection(i, a, b)
    ideal_quotient_mul_projection(i, a - b, c)
    ideal_quotient_mul_projection(i, a, c)
    ideal_quotient_mul_projection(i, b, c)
    ideal_quotient_sub_projection(i, a * c, b * c)
    quotient_over_mk(ideal_quotient_relation(i), (a - b) * c) =
        quotient_over_mk(ideal_quotient_relation(i), a * c - b * c)
}

/// Quotient subtraction reassociates with quotient addition on the left.
theorem ideal_quotient_add_sub_assoc[R: CommRing](i: Ideal[R], a: R, b: R, c: R) {
    ideal_quotient_sub(i,
        ideal_quotient_add(i,
            quotient_over_mk(ideal_quotient_relation(i), a),
            quotient_over_mk(ideal_quotient_relation(i), b)),
        quotient_over_mk(ideal_quotient_relation(i), c)) =
    ideal_quotient_add(i,
        quotient_over_mk(ideal_quotient_relation(i), a),
        ideal_quotient_sub(i,
            quotient_over_mk(ideal_quotient_relation(i), b),
            quotient_over_mk(ideal_quotient_relation(i), c)))
} by {
    ideal_quotient_add_projection(i, a, b)
    ideal_quotient_sub_projection(i, a + b, c)
    ideal_quotient_sub(i,
        ideal_quotient_add(i,
            quotient_over_mk(ideal_quotient_relation(i), a),
            quotient_over_mk(ideal_quotient_relation(i), b)),
        quotient_over_mk(ideal_quotient_relation(i), c)) =
        quotient_over_mk(ideal_quotient_relation(i), (a + b) - c)
    ideal_quotient_sub_projection(i, b, c)
    ideal_quotient_add_projection(i, a, b - c)
    ideal_quotient_add(i,
        quotient_over_mk(ideal_quotient_relation(i), a),
        ideal_quotient_sub(i,
            quotient_over_mk(ideal_quotient_relation(i), b),
            quotient_over_mk(ideal_quotient_relation(i), c))) =
        quotient_over_mk(ideal_quotient_relation(i), a + (b - c))
    (a + b) - c = a + (b - c)
}

/// Quotient addition reassociates with quotient subtraction on the left.
theorem ideal_quotient_sub_add_assoc[R: CommRing](i: Ideal[R], a: R, b: R, c: R) {
    ideal_quotient_add(i,
        ideal_quotient_sub(i,
            quotient_over_mk(ideal_quotient_relation(i), a),
            quotient_over_mk(ideal_quotient_relation(i), b)),
        quotient_over_mk(ideal_quotient_relation(i), c)) =
    ideal_quotient_sub(i,
        quotient_over_mk(ideal_quotient_relation(i), a),
        ideal_quotient_sub(i,
            quotient_over_mk(ideal_quotient_relation(i), b),
            quotient_over_mk(ideal_quotient_relation(i), c)))
} by {
    ideal_quotient_sub_projection(i, a, b)
    ideal_quotient_add_projection(i, a - b, c)
    ideal_quotient_sub_projection(i, b, c)
    ideal_quotient_sub_projection(i, a, b - c)
    a - (b - c) = a + (-b + c)
    quotient_over_mk(ideal_quotient_relation(i), (a - b) + c) =
        quotient_over_mk(ideal_quotient_relation(i), a - (b - c))
}

/// Quotient subtraction reassociates nested right subtraction as addition.
theorem ideal_quotient_sub_sub_assoc[R: CommRing](i: Ideal[R], a: R, b: R, c: R) {
    ideal_quotient_sub(i,
        ideal_quotient_sub(i,
            quotient_over_mk(ideal_quotient_relation(i), a),
            quotient_over_mk(ideal_quotient_relation(i), b)),
        quotient_over_mk(ideal_quotient_relation(i), c)) =
    ideal_quotient_sub(i,
        quotient_over_mk(ideal_quotient_relation(i), a),
        ideal_quotient_add(i,
            quotient_over_mk(ideal_quotient_relation(i), b),
            quotient_over_mk(ideal_quotient_relation(i), c)))
} by {
    ideal_quotient_sub_projection(i, a, b)
    ideal_quotient_sub_projection(i, a - b, c)
    ideal_quotient_add_projection(i, b, c)
    ideal_quotient_sub_projection(i, a, b + c)
    a + (-b + -c) = (a - b) - c
    quotient_over_mk(ideal_quotient_relation(i), (a - b) - c) =
        quotient_over_mk(ideal_quotient_relation(i), a - (b + c))
}

/// Consecutive quotient differences telescope through a middle representative.
theorem ideal_quotient_sub_add_sub_cancel[R: CommRing](i: Ideal[R], a: R, b: R, c: R) {
    ideal_quotient_add(i,
        ideal_quotient_sub(i,
            quotient_over_mk(ideal_quotient_relation(i), a),
            quotient_over_mk(ideal_quotient_relation(i), b)),
        ideal_quotient_sub(i,
            quotient_over_mk(ideal_quotient_relation(i), b),
            quotient_over_mk(ideal_quotient_relation(i), c))) =
    ideal_quotient_sub(i,
        quotient_over_mk(ideal_quotient_relation(i), a),
        quotient_over_mk(ideal_quotient_relation(i), c))
} by {
    ideal_quotient_sub_projection(i, a, b)
    ideal_quotient_sub_projection(i, b, c)
    ideal_quotient_add_projection(i, a - b, b - c)
    ideal_quotient_sub_projection(i, a, c)
    quotient_over_mk(ideal_quotient_relation(i), (a - b) + (b - c)) =
        quotient_over_mk(ideal_quotient_relation(i), a - c)
}

/// Subtracting quotient sums with a common left summand cancels that summand.
theorem ideal_quotient_add_sub_cancel_left[R: CommRing](i: Ideal[R], a: R, b: R, c: R) {
    ideal_quotient_sub(i,
        ideal_quotient_add(i,
            quotient_over_mk(ideal_quotient_relation(i), a),
            quotient_over_mk(ideal_quotient_relation(i), b)),
        ideal_quotient_add(i,
            quotient_over_mk(ideal_quotient_relation(i), a),
            quotient_over_mk(ideal_quotient_relation(i), c))) =
    ideal_quotient_sub(i,
        quotient_over_mk(ideal_quotient_relation(i), b),
        quotient_over_mk(ideal_quotient_relation(i), c))
} by {
    ideal_quotient_add_projection(i, a, b)
    ideal_quotient_add_projection(i, a, c)
    ideal_quotient_sub_projection(i, a + b, a + c)
    ideal_quotient_sub_projection(i, b, c)
    (a + b) - (a + c) = b - c
}

/// Subtracting quotient sums with a common right summand cancels that summand.
theorem ideal_quotient_add_sub_cancel_right[R: CommRing](i: Ideal[R], a: R, b: R, c: R) {
    ideal_quotient_sub(i,
        ideal_quotient_add(i,
            quotient_over_mk(ideal_quotient_relation(i), a),
            quotient_over_mk(ideal_quotient_relation(i), c)),
        ideal_quotient_add(i,
            quotient_over_mk(ideal_quotient_relation(i), b),
            quotient_over_mk(ideal_quotient_relation(i), c))) =
    ideal_quotient_sub(i,
        quotient_over_mk(ideal_quotient_relation(i), a),
        quotient_over_mk(ideal_quotient_relation(i), b))
} by {
    ideal_quotient_add_projection(i, a, c)
    ideal_quotient_add_projection(i, b, c)
    ideal_quotient_sub_projection(i, a + c, b + c)
    ideal_quotient_sub(i,
        ideal_quotient_add(i,
            quotient_over_mk(ideal_quotient_relation(i), a),
            quotient_over_mk(ideal_quotient_relation(i), c)),
        ideal_quotient_add(i,
            quotient_over_mk(ideal_quotient_relation(i), b),
            quotient_over_mk(ideal_quotient_relation(i), c))) =
        quotient_over_mk(ideal_quotient_relation(i), (a + c) - (b + c))
    ideal_quotient_sub_projection(i, a, b)
    ideal_quotient_sub(i,
        quotient_over_mk(ideal_quotient_relation(i), a),
        quotient_over_mk(ideal_quotient_relation(i), b)) =
        quotient_over_mk(ideal_quotient_relation(i), a - b)
    (a + c) - (b + c) = (a + c) + -(b + c)
    -(b + c) = -b + -c
    c + -c = R.0
    quotient_over_mk(ideal_quotient_relation(i), (a + c) - (b + c)) =
        quotient_over_mk(ideal_quotient_relation(i), a - b)
}

/// The negation of quotient zero is quotient zero.
theorem ideal_quotient_neg_zero[R: CommRing](i: Ideal[R]) {
    ideal_quotient_neg(i, ideal_quotient_zero(i)) = ideal_quotient_zero(i)
} by {
    ideal_quotient_zero_projection(i)
    ideal_quotient_neg_projection(i, R.0)
    quotient_over_mk(ideal_quotient_relation(i), -R.0) =
        quotient_over_mk(ideal_quotient_relation(i), R.0)
}

/// Negating twice fixes a projected ideal quotient representative.
theorem ideal_quotient_neg_neg[R: CommRing](i: Ideal[R], a: R) {
    ideal_quotient_neg(i,
        ideal_quotient_neg(i, quotient_over_mk(ideal_quotient_relation(i), a))) =
        quotient_over_mk(ideal_quotient_relation(i), a)
} by {
    ideal_quotient_neg_projection(i, a)
    ideal_quotient_neg_projection(i, -a)
    quotient_over_mk(ideal_quotient_relation(i), --a) =
        quotient_over_mk(ideal_quotient_relation(i), a)
}

/// Multiplication by quotient zero on the left is quotient zero.
theorem ideal_quotient_zero_mul[R: CommRing](i: Ideal[R], a: R) {
    ideal_quotient_mul(i,
        ideal_quotient_zero(i),
        quotient_over_mk(ideal_quotient_relation(i), a)) =
        ideal_quotient_zero(i)
} by {
    ideal_quotient_mul_projection(i, R.0, a)
    R.0 * a = R.0
}

/// Multiplication by quotient zero on the right is quotient zero.
theorem ideal_quotient_mul_zero[R: CommRing](i: Ideal[R], a: R) {
    ideal_quotient_mul(i,
        quotient_over_mk(ideal_quotient_relation(i), a),
        ideal_quotient_zero(i)) =
        ideal_quotient_zero(i)
} by {
    ideal_quotient_mul_projection(i, a, R.0)
    a * R.0 = R.0
}

/// A quotient product with a negated left representative is the negation of the quotient product.
theorem ideal_quotient_neg_mul[R: CommRing](i: Ideal[R], a: R, b: R) {
    ideal_quotient_mul(i,
        ideal_quotient_neg(i, quotient_over_mk(ideal_quotient_relation(i), a)),
        quotient_over_mk(ideal_quotient_relation(i), b)) =
    ideal_quotient_neg(i,
        ideal_quotient_mul(i,
            quotient_over_mk(ideal_quotient_relation(i), a),
            quotient_over_mk(ideal_quotient_relation(i), b)))
} by {
    ideal_quotient_neg_projection(i, a)
    ideal_quotient_mul_projection(i, -a, b)

    ideal_quotient_mul_projection(i, a, b)
    ideal_quotient_neg_projection(i, a * b)
    quotient_over_mk(ideal_quotient_relation(i), -a * b) =
        quotient_over_mk(ideal_quotient_relation(i), -(a * b))
}

/// A quotient product with a negated right representative is the negation of the quotient product.
theorem ideal_quotient_mul_neg[R: CommRing](i: Ideal[R], a: R, b: R) {
    ideal_quotient_mul(i,
        quotient_over_mk(ideal_quotient_relation(i), a),
        ideal_quotient_neg(i, quotient_over_mk(ideal_quotient_relation(i), b))) =
    ideal_quotient_neg(i,
        ideal_quotient_mul(i,
            quotient_over_mk(ideal_quotient_relation(i), a),
            quotient_over_mk(ideal_quotient_relation(i), b)))
} by {
    ideal_quotient_neg_projection(i, b)
    ideal_quotient_mul_projection(i, a, -b)

    ideal_quotient_mul_projection(i, a, b)
    ideal_quotient_neg_projection(i, a * b)
    quotient_over_mk(ideal_quotient_relation(i), a * -b) =
        quotient_over_mk(ideal_quotient_relation(i), -(a * b))
}

/// Negation distributes over quotient addition of projected representatives.
theorem ideal_quotient_neg_add_projection[R: CommRing](i: Ideal[R], a: R, b: R) {
    ideal_quotient_neg(i,
        ideal_quotient_add(i,
            quotient_over_mk(ideal_quotient_relation(i), a),
            quotient_over_mk(ideal_quotient_relation(i), b))) =
    ideal_quotient_add(i,
        ideal_quotient_neg(i, quotient_over_mk(ideal_quotient_relation(i), a)),
        ideal_quotient_neg(i, quotient_over_mk(ideal_quotient_relation(i), b)))
} by {
    ideal_quotient_add_projection(i, a, b)
    ideal_quotient_neg_projection(i, a + b)

    ideal_quotient_neg_projection(i, a)
    ideal_quotient_neg_projection(i, b)
    ideal_quotient_add_projection(i, -a, -b)
    quotient_over_mk(ideal_quotient_relation(i), -(a + b)) =
        quotient_over_mk(ideal_quotient_relation(i), -a + -b)
}

/// Negation sends quotient subtraction to the opposite quotient subtraction.
theorem ideal_quotient_neg_sub_projection[R: CommRing](i: Ideal[R], a: R, b: R) {
    ideal_quotient_neg(i,
        ideal_quotient_sub(i,
            quotient_over_mk(ideal_quotient_relation(i), a),
            quotient_over_mk(ideal_quotient_relation(i), b))) =
    ideal_quotient_sub(i,
        quotient_over_mk(ideal_quotient_relation(i), b),
        quotient_over_mk(ideal_quotient_relation(i), a))
} by {
    ideal_quotient_sub_projection(i, a, b)
    ideal_quotient_neg_projection(i, a - b)
    ideal_quotient_sub_projection(i, b, a)
    ideal_quotient_neg_sub_eq_swap(a, b)
}

/// Subtracting a negated quotient representative is quotient addition.
theorem ideal_quotient_sub_neg_projection[R: CommRing](i: Ideal[R], a: R, b: R) {
    ideal_quotient_sub(i,
        quotient_over_mk(ideal_quotient_relation(i), a),
        ideal_quotient_neg(i, quotient_over_mk(ideal_quotient_relation(i), b))) =
    ideal_quotient_add(i,
        quotient_over_mk(ideal_quotient_relation(i), a),
        quotient_over_mk(ideal_quotient_relation(i), b))
} by {
    ideal_quotient_neg_projection(i, b)
    ideal_quotient_sub_projection(i, a, -b)
    ideal_quotient_add_projection(i, a, b)
}

/// The product of two negated quotient representatives is the quotient product.
theorem ideal_quotient_neg_mul_neg[R: CommRing](i: Ideal[R], a: R, b: R) {
    ideal_quotient_mul(i,
        ideal_quotient_neg(i, quotient_over_mk(ideal_quotient_relation(i), a)),
        ideal_quotient_neg(i, quotient_over_mk(ideal_quotient_relation(i), b))) =
    ideal_quotient_mul(i,
        quotient_over_mk(ideal_quotient_relation(i), a),
        quotient_over_mk(ideal_quotient_relation(i), b))
} by {
    ideal_quotient_neg_projection(i, a)
    ideal_quotient_neg_projection(i, b)
    ideal_quotient_mul_projection(i, -a, -b)
    ideal_quotient_mul_projection(i, a, b)
    quotient_over_mk(ideal_quotient_relation(i), -a * -b) =
        quotient_over_mk(ideal_quotient_relation(i), a * b)
}

/// Adding quotient zero on the left fixes a projected representative.
theorem ideal_quotient_project_zero_add[R: CommRing](i: Ideal[R], a: R) {
    ideal_quotient_add(i, ideal_quotient_zero(i), ideal_quotient_project(i, a)) =
        ideal_quotient_project(i, a)
} by {
    ideal_quotient_zero_add(i, a)
}

/// Adding quotient zero on the right fixes a projected representative.
theorem ideal_quotient_project_add_zero[R: CommRing](i: Ideal[R], a: R) {
    ideal_quotient_add(i, ideal_quotient_project(i, a), ideal_quotient_zero(i)) =
        ideal_quotient_project(i, a)
} by {
    ideal_quotient_add_zero(i, a)
}

/// Multiplying by quotient one on the left fixes a projected representative.
theorem ideal_quotient_project_one_mul[R: CommRing](i: Ideal[R], a: R) {
    ideal_quotient_mul(i, ideal_quotient_one(i), ideal_quotient_project(i, a)) =
        ideal_quotient_project(i, a)
} by {
    ideal_quotient_one_mul(i, a)
}

/// Multiplying by quotient one on the right fixes a projected representative.
theorem ideal_quotient_project_mul_one[R: CommRing](i: Ideal[R], a: R) {
    ideal_quotient_mul(i, ideal_quotient_project(i, a), ideal_quotient_one(i)) =
        ideal_quotient_project(i, a)
} by {
    ideal_quotient_mul_one(i, a)
}

/// Multiplying by quotient zero on the left gives quotient zero.
theorem ideal_quotient_project_zero_mul[R: CommRing](i: Ideal[R], a: R) {
    ideal_quotient_mul(i, ideal_quotient_zero(i), ideal_quotient_project(i, a)) =
        ideal_quotient_zero(i)
} by {
    ideal_quotient_zero_mul(i, a)
}

/// Multiplying by quotient zero on the right gives quotient zero.
theorem ideal_quotient_project_mul_zero[R: CommRing](i: Ideal[R], a: R) {
    ideal_quotient_mul(i, ideal_quotient_project(i, a), ideal_quotient_zero(i)) =
        ideal_quotient_zero(i)
} by {
    ideal_quotient_mul_zero(i, a)
}

/// Subtracting quotient zero on the right fixes a projected representative.
theorem ideal_quotient_project_sub_zero[R: CommRing](i: Ideal[R], a: R) {
    ideal_quotient_sub(i, ideal_quotient_project(i, a), ideal_quotient_zero(i)) =
        ideal_quotient_project(i, a)
} by {
    ideal_quotient_sub_zero(i, a)
}

/// Subtracting a projected representative from quotient zero gives its quotient negation.
theorem ideal_quotient_project_zero_sub[R: CommRing](i: Ideal[R], a: R) {
    ideal_quotient_sub(i, ideal_quotient_zero(i), ideal_quotient_project(i, a)) =
        ideal_quotient_neg(i, ideal_quotient_project(i, a))
} by {
    ideal_quotient_zero_sub(i, a)
    ideal_quotient_neg_projection(i, a)
}

/// A projected representative subtracted from itself is quotient zero.
theorem ideal_quotient_project_sub_self[R: CommRing](i: Ideal[R], a: R) {
    ideal_quotient_sub(i, ideal_quotient_project(i, a), ideal_quotient_project(i, a)) =
        ideal_quotient_zero(i)
} by {
    ideal_quotient_sub_self(i, a)
}

/// Quotient subtraction of projected representatives is addition of the quotient negation.
theorem ideal_quotient_project_sub_eq_add_neg[R: CommRing](i: Ideal[R], a: R, b: R) {
    ideal_quotient_sub(i, ideal_quotient_project(i, a), ideal_quotient_project(i, b)) =
    ideal_quotient_add(i, ideal_quotient_project(i, a),
        ideal_quotient_neg(i, ideal_quotient_project(i, b)))
} by {
    ideal_quotient_sub_eq_add_neg(i, a, b)
}

/// Quotient addition of projected representatives is commutative.
theorem ideal_quotient_project_add_comm[R: CommRing](i: Ideal[R], a: R, b: R) {
    ideal_quotient_add(i, ideal_quotient_project(i, a), ideal_quotient_project(i, b)) =
    ideal_quotient_add(i, ideal_quotient_project(i, b), ideal_quotient_project(i, a))
} by {
    ideal_quotient_add_comm(i, a, b)
}

/// Quotient multiplication of projected representatives is commutative.
theorem ideal_quotient_project_mul_comm[R: CommRing](i: Ideal[R], a: R, b: R) {
    ideal_quotient_mul(i, ideal_quotient_project(i, a), ideal_quotient_project(i, b)) =
    ideal_quotient_mul(i, ideal_quotient_project(i, b), ideal_quotient_project(i, a))
} by {
    ideal_quotient_mul_comm(i, a, b)
}

/// Quotient addition of projected representatives is associative.
theorem ideal_quotient_project_add_assoc[R: CommRing](i: Ideal[R], a: R, b: R, c: R) {
    ideal_quotient_add(i,
        ideal_quotient_add(i, ideal_quotient_project(i, a), ideal_quotient_project(i, b)),
        ideal_quotient_project(i, c)) =
    ideal_quotient_add(i,
        ideal_quotient_project(i, a),
        ideal_quotient_add(i, ideal_quotient_project(i, b), ideal_quotient_project(i, c)))
} by {
    ideal_quotient_add_assoc(i, a, b, c)
}

/// Quotient multiplication of projected representatives is associative.
theorem ideal_quotient_project_mul_assoc[R: CommRing](i: Ideal[R], a: R, b: R, c: R) {
    ideal_quotient_mul(i,
        ideal_quotient_mul(i, ideal_quotient_project(i, a), ideal_quotient_project(i, b)),
        ideal_quotient_project(i, c)) =
    ideal_quotient_mul(i,
        ideal_quotient_project(i, a),
        ideal_quotient_mul(i, ideal_quotient_project(i, b), ideal_quotient_project(i, c)))
} by {
    ideal_quotient_mul_assoc(i, a, b, c)
}

/// Quotient multiplication distributes over projected quotient addition on the left.
theorem ideal_quotient_project_mul_add[R: CommRing](i: Ideal[R], a: R, b: R, c: R) {
    ideal_quotient_mul(i,
        ideal_quotient_project(i, a),
        ideal_quotient_add(i, ideal_quotient_project(i, b), ideal_quotient_project(i, c))) =
    ideal_quotient_add(i,
        ideal_quotient_mul(i, ideal_quotient_project(i, a), ideal_quotient_project(i, b)),
        ideal_quotient_mul(i, ideal_quotient_project(i, a), ideal_quotient_project(i, c)))
} by {
    ideal_quotient_mul_add(i, a, b, c)
}

/// Quotient multiplication distributes over projected quotient addition on the right.
theorem ideal_quotient_project_add_mul[R: CommRing](i: Ideal[R], a: R, b: R, c: R) {
    ideal_quotient_mul(i,
        ideal_quotient_add(i, ideal_quotient_project(i, a), ideal_quotient_project(i, b)),
        ideal_quotient_project(i, c)) =
    ideal_quotient_add(i,
        ideal_quotient_mul(i, ideal_quotient_project(i, a), ideal_quotient_project(i, c)),
        ideal_quotient_mul(i, ideal_quotient_project(i, b), ideal_quotient_project(i, c)))
} by {
    ideal_quotient_add_mul(i, a, b, c)
}

/// Quotient multiplication distributes over projected quotient subtraction on the left.
theorem ideal_quotient_project_mul_sub[R: CommRing](i: Ideal[R], a: R, b: R, c: R) {
    ideal_quotient_mul(i,
        ideal_quotient_project(i, a),
        ideal_quotient_sub(i, ideal_quotient_project(i, b), ideal_quotient_project(i, c))) =
    ideal_quotient_sub(i,
        ideal_quotient_mul(i, ideal_quotient_project(i, a), ideal_quotient_project(i, b)),
        ideal_quotient_mul(i, ideal_quotient_project(i, a), ideal_quotient_project(i, c)))
} by {
    ideal_quotient_mul_sub(i, a, b, c)
}

/// Quotient multiplication distributes over projected quotient subtraction on the right.
theorem ideal_quotient_project_sub_mul[R: CommRing](i: Ideal[R], a: R, b: R, c: R) {
    ideal_quotient_mul(i,
        ideal_quotient_sub(i, ideal_quotient_project(i, a), ideal_quotient_project(i, b)),
        ideal_quotient_project(i, c)) =
    ideal_quotient_sub(i,
        ideal_quotient_mul(i, ideal_quotient_project(i, a), ideal_quotient_project(i, c)),
        ideal_quotient_mul(i, ideal_quotient_project(i, b), ideal_quotient_project(i, c)))
} by {
    ideal_quotient_sub_mul(i, a, b, c)
}

/// Projected quotient subtraction reassociates with quotient addition on the left.
theorem ideal_quotient_project_add_sub_assoc[R: CommRing](i: Ideal[R], a: R, b: R, c: R) {
    ideal_quotient_sub(i,
        ideal_quotient_add(i, ideal_quotient_project(i, a), ideal_quotient_project(i, b)),
        ideal_quotient_project(i, c)) =
    ideal_quotient_add(i,
        ideal_quotient_project(i, a),
        ideal_quotient_sub(i, ideal_quotient_project(i, b), ideal_quotient_project(i, c)))
} by {
    ideal_quotient_add_sub_assoc(i, a, b, c)
}

/// Projected quotient addition reassociates with quotient subtraction on the left.
theorem ideal_quotient_project_sub_add_assoc[R: CommRing](i: Ideal[R], a: R, b: R, c: R) {
    ideal_quotient_add(i,
        ideal_quotient_sub(i, ideal_quotient_project(i, a), ideal_quotient_project(i, b)),
        ideal_quotient_project(i, c)) =
    ideal_quotient_sub(i,
        ideal_quotient_project(i, a),
        ideal_quotient_sub(i, ideal_quotient_project(i, b), ideal_quotient_project(i, c)))
} by {
    ideal_quotient_sub_add_assoc(i, a, b, c)
}

/// Projected quotient subtraction reassociates nested right subtraction as addition.
theorem ideal_quotient_project_sub_sub_assoc[R: CommRing](i: Ideal[R], a: R, b: R, c: R) {
    ideal_quotient_sub(i,
        ideal_quotient_sub(i, ideal_quotient_project(i, a), ideal_quotient_project(i, b)),
        ideal_quotient_project(i, c)) =
    ideal_quotient_sub(i,
        ideal_quotient_project(i, a),
        ideal_quotient_add(i, ideal_quotient_project(i, b), ideal_quotient_project(i, c)))
} by {
    ideal_quotient_sub_sub_assoc(i, a, b, c)
}

/// Consecutive projected quotient differences telescope through a middle representative.
theorem ideal_quotient_project_sub_add_sub_cancel[R: CommRing](i: Ideal[R], a: R, b: R, c: R) {
    ideal_quotient_add(i,
        ideal_quotient_sub(i, ideal_quotient_project(i, a), ideal_quotient_project(i, b)),
        ideal_quotient_sub(i, ideal_quotient_project(i, b), ideal_quotient_project(i, c))) =
    ideal_quotient_sub(i, ideal_quotient_project(i, a), ideal_quotient_project(i, c))
} by {
    ideal_quotient_sub_add_sub_cancel(i, a, b, c)
}

/// Subtracting projected quotient sums with a common left summand cancels that summand.
theorem ideal_quotient_project_add_sub_cancel_left[R: CommRing](i: Ideal[R], a: R, b: R, c: R) {
    ideal_quotient_sub(i,
        ideal_quotient_add(i, ideal_quotient_project(i, a), ideal_quotient_project(i, b)),
        ideal_quotient_add(i, ideal_quotient_project(i, a), ideal_quotient_project(i, c))) =
    ideal_quotient_sub(i, ideal_quotient_project(i, b), ideal_quotient_project(i, c))
} by {
    ideal_quotient_add_sub_cancel_left(i, a, b, c)
}

/// Subtracting projected quotient sums with a common right summand cancels that summand.
theorem ideal_quotient_project_add_sub_cancel_right[R: CommRing](i: Ideal[R], a: R, b: R, c: R) {
    ideal_quotient_sub(i,
        ideal_quotient_add(i, ideal_quotient_project(i, a), ideal_quotient_project(i, c)),
        ideal_quotient_add(i, ideal_quotient_project(i, b), ideal_quotient_project(i, c))) =
    ideal_quotient_sub(i, ideal_quotient_project(i, a), ideal_quotient_project(i, b))
} by {
    ideal_quotient_add_sub_cancel_right(i, a, b, c)
}

/// Quotient negation sends quotient zero to quotient zero.
theorem ideal_quotient_project_neg_zero[R: CommRing](i: Ideal[R]) {
    ideal_quotient_neg(i, ideal_quotient_zero(i)) = ideal_quotient_zero(i)
} by {
    ideal_quotient_neg_zero(i)
}

/// Quotient negation is involutive on projected representatives.
theorem ideal_quotient_project_neg_neg[R: CommRing](i: Ideal[R], a: R) {
    ideal_quotient_neg(i, ideal_quotient_neg(i, ideal_quotient_project(i, a))) =
        ideal_quotient_project(i, a)
} by {
    ideal_quotient_neg_neg(i, a)
}

/// The quotient negation is a left inverse for projected addition.
theorem ideal_quotient_project_neg_add[R: CommRing](i: Ideal[R], a: R) {
    ideal_quotient_add(i,
        ideal_quotient_neg(i, ideal_quotient_project(i, a)),
        ideal_quotient_project(i, a)) =
        ideal_quotient_zero(i)
} by {
    ideal_quotient_neg_add(i, a)
}

/// The quotient negation is a right inverse for projected addition.
theorem ideal_quotient_project_add_neg[R: CommRing](i: Ideal[R], a: R) {
    ideal_quotient_add(i,
        ideal_quotient_project(i, a),
        ideal_quotient_neg(i, ideal_quotient_project(i, a))) =
        ideal_quotient_zero(i)
} by {
    ideal_quotient_add_neg(i, a)
}

/// Negation distributes over quotient addition of projected representatives.
theorem ideal_quotient_project_neg_add_projection[R: CommRing](i: Ideal[R], a: R, b: R) {
    ideal_quotient_neg(i,
        ideal_quotient_add(i, ideal_quotient_project(i, a), ideal_quotient_project(i, b))) =
    ideal_quotient_add(i,
        ideal_quotient_neg(i, ideal_quotient_project(i, a)),
        ideal_quotient_neg(i, ideal_quotient_project(i, b)))
} by {
    ideal_quotient_neg_add_projection(i, a, b)
}

/// Negation sends quotient subtraction of projected representatives to the opposite subtraction.
theorem ideal_quotient_project_neg_sub_projection[R: CommRing](i: Ideal[R], a: R, b: R) {
    ideal_quotient_neg(i,
        ideal_quotient_sub(i, ideal_quotient_project(i, a), ideal_quotient_project(i, b))) =
    ideal_quotient_sub(i, ideal_quotient_project(i, b), ideal_quotient_project(i, a))
} by {
    ideal_quotient_neg_sub_projection(i, a, b)
}

/// Subtracting a negated projected representative is quotient addition.
theorem ideal_quotient_project_sub_neg_projection[R: CommRing](i: Ideal[R], a: R, b: R) {
    ideal_quotient_sub(i,
        ideal_quotient_project(i, a),
        ideal_quotient_neg(i, ideal_quotient_project(i, b))) =
    ideal_quotient_add(i, ideal_quotient_project(i, a), ideal_quotient_project(i, b))
} by {
    ideal_quotient_sub_neg_projection(i, a, b)
}

/// A quotient product with a negated left projected representative is the negation of the quotient product.
theorem ideal_quotient_project_neg_mul[R: CommRing](i: Ideal[R], a: R, b: R) {
    ideal_quotient_mul(i,
        ideal_quotient_neg(i, ideal_quotient_project(i, a)),
        ideal_quotient_project(i, b)) =
    ideal_quotient_neg(i,
        ideal_quotient_mul(i, ideal_quotient_project(i, a), ideal_quotient_project(i, b)))
} by {
    ideal_quotient_neg_mul(i, a, b)
}

/// A quotient product with a negated right projected representative is the negation of the quotient product.
theorem ideal_quotient_project_mul_neg[R: CommRing](i: Ideal[R], a: R, b: R) {
    ideal_quotient_mul(i,
        ideal_quotient_project(i, a),
        ideal_quotient_neg(i, ideal_quotient_project(i, b))) =
    ideal_quotient_neg(i,
        ideal_quotient_mul(i, ideal_quotient_project(i, a), ideal_quotient_project(i, b)))
} by {
    ideal_quotient_mul_neg(i, a, b)
}

/// The product of two negated projected representatives is the quotient product.
theorem ideal_quotient_project_neg_mul_neg[R: CommRing](i: Ideal[R], a: R, b: R) {
    ideal_quotient_mul(i,
        ideal_quotient_neg(i, ideal_quotient_project(i, a)),
        ideal_quotient_neg(i, ideal_quotient_project(i, b))) =
    ideal_quotient_mul(i, ideal_quotient_project(i, a), ideal_quotient_project(i, b))
} by {
    ideal_quotient_neg_mul_neg(i, a, b)
}

/// Powers on the ideal quotient `R/I`.
define ideal_quotient_pow[R: CommRing](
    i: Ideal[R], q: QuotientOver[R], n: Nat
) -> QuotientOver[R] {
    match n {
        Nat.zero {
            ideal_quotient_one(i)
        }
        Nat.suc(k) {
            ideal_quotient_mul(i, q, ideal_quotient_pow(i, q, k))
        }
    }
}

/// The zeroth quotient power is quotient one.
theorem ideal_quotient_pow_zero[R: CommRing](i: Ideal[R], q: QuotientOver[R]) {
    ideal_quotient_pow(i, q, Nat.0) = ideal_quotient_one(i)
}

/// The successor quotient power multiplies once more by the base.
theorem ideal_quotient_pow_suc[R: CommRing](i: Ideal[R], q: QuotientOver[R], n: Nat) {
    ideal_quotient_pow(i, q, n.suc) =
        ideal_quotient_mul(i, q, ideal_quotient_pow(i, q, n))
}

/// The zeroth power of a projected representative is the projected zeroth power.
theorem ideal_quotient_pow_mk_zero[R: CommRing](i: Ideal[R], a: R) {
    ideal_quotient_pow(i, quotient_over_mk(ideal_quotient_relation(i), a), Nat.0) =
        quotient_over_mk(ideal_quotient_relation(i), a.pow(Nat.0))
} by {
    ideal_quotient_pow_zero(i, quotient_over_mk(ideal_quotient_relation(i), a))
    ideal_quotient_one_projection(i)
}

/// Powers of projected representatives agree with projected powers.
theorem ideal_quotient_pow_mk[R: CommRing](i: Ideal[R], a: R, n: Nat) {
    ideal_quotient_pow(i, quotient_over_mk(ideal_quotient_relation(i), a), n) =
        quotient_over_mk(ideal_quotient_relation(i), a.pow(n))
} by {
    define p(k: Nat) -> Bool {
        ideal_quotient_pow(i, quotient_over_mk(ideal_quotient_relation(i), a), k) =
            quotient_over_mk(ideal_quotient_relation(i), a.pow(k))
    }
    ideal_quotient_pow_mk_zero(i, a)
    p(Nat.0)
    forall(k: Nat) {
        if p(k) {
            ideal_quotient_pow_suc(
                i, quotient_over_mk(ideal_quotient_relation(i), a), k)
            ideal_quotient_mul(i,
                quotient_over_mk(ideal_quotient_relation(i), a),
                ideal_quotient_pow(i,
                    quotient_over_mk(ideal_quotient_relation(i), a), k)) =
                ideal_quotient_mul(i,
                    quotient_over_mk(ideal_quotient_relation(i), a),
                    quotient_over_mk(ideal_quotient_relation(i), a.pow(k)))
            ideal_quotient_mul_projection(i, a, a.pow(k))
            p(k.suc)
        }
    }
    p(n)
}

/// The canonical projection preserves natural powers.
theorem ideal_quotient_project_pow[R: CommRing](i: Ideal[R], a: R, n: Nat) {
    ideal_quotient_project(i, a.pow(n)) =
    ideal_quotient_pow(i, ideal_quotient_project(i, a), n)
} by {
    ideal_quotient_pow_mk(i, a, n)
}

/// The first power of a projected representative is the projected representative.
theorem ideal_quotient_pow_one[R: CommRing](i: Ideal[R], a: R) {
    ideal_quotient_pow(i, quotient_over_mk(ideal_quotient_relation(i), a), Nat.1) =
        quotient_over_mk(ideal_quotient_relation(i), a)
} by {
    ideal_quotient_pow_mk(i, a, Nat.1)
}

/// Quotient one raised to a natural power is quotient one.
theorem ideal_quotient_one_pow[R: CommRing](i: Ideal[R], n: Nat) {
    ideal_quotient_pow(i, ideal_quotient_one(i), n) = ideal_quotient_one(i)
} by {
    define p(k: Nat) -> Bool {
        ideal_quotient_pow(i, ideal_quotient_one(i), k) = ideal_quotient_one(i)
    }
    ideal_quotient_pow_zero(i, ideal_quotient_one(i))
    p(Nat.0)
    forall(k: Nat) {
        if p(k) {
            ideal_quotient_pow_suc(i, ideal_quotient_one(i), k)
            ideal_quotient_mul(i, ideal_quotient_one(i),
                ideal_quotient_pow(i, ideal_quotient_one(i), k)) =
                ideal_quotient_mul(i, ideal_quotient_one(i), ideal_quotient_one(i))
            ideal_quotient_mul_projection(i, R.1, R.1)
            ideal_quotient_one_projection(i)
            p(k.suc)
        }
    }
    p(n)
}

/// Multiplying two projected powers adds their exponents.
theorem ideal_quotient_pow_add[R: CommRing](i: Ideal[R], a: R, n: Nat, m: Nat) {
    ideal_quotient_mul(i,
        ideal_quotient_pow(i, quotient_over_mk(ideal_quotient_relation(i), a), n),
        ideal_quotient_pow(i, quotient_over_mk(ideal_quotient_relation(i), a), m)) =
    ideal_quotient_pow(i, quotient_over_mk(ideal_quotient_relation(i), a), n + m)
} by {
    ideal_quotient_pow_mk(i, a, n)
    ideal_quotient_pow_mk(i, a, m)
    ideal_quotient_mul_projection(i, a.pow(n), a.pow(m))
    ideal_quotient_pow_mk(i, a, n + m)
    quotient_over_mk(ideal_quotient_relation(i), a.pow(n) * a.pow(m)) =
        quotient_over_mk(ideal_quotient_relation(i), a.pow(n + m))
}

/// Raising a projected power to a power multiplies the exponents.
theorem ideal_quotient_pow_pow[R: CommRing](i: Ideal[R], a: R, n: Nat, m: Nat) {
    ideal_quotient_pow(i,
        ideal_quotient_pow(i, quotient_over_mk(ideal_quotient_relation(i), a), n),
        m) =
    ideal_quotient_pow(i, quotient_over_mk(ideal_quotient_relation(i), a), n * m)
} by {
    ideal_quotient_pow_mk(i, a, n)
    ideal_quotient_pow_mk(i, a.pow(n), m)
    ideal_quotient_pow_mk(i, a, n * m)
}

/// The first quotient power of a projected representative is the projected representative.
theorem ideal_quotient_project_pow_one[R: CommRing](i: Ideal[R], a: R) {
    ideal_quotient_pow(i, ideal_quotient_project(i, a), Nat.1) =
        ideal_quotient_project(i, a)
} by {
    ideal_quotient_pow_one(i, a)
}

/// Quotient one raised to a natural power is quotient one.
theorem ideal_quotient_project_one_pow[R: CommRing](i: Ideal[R], n: Nat) {
    ideal_quotient_pow(i, ideal_quotient_one(i), n) = ideal_quotient_one(i)
} by {
    ideal_quotient_one_pow(i, n)
}

/// Multiplying projected quotient powers adds their exponents.
theorem ideal_quotient_project_pow_add[R: CommRing](i: Ideal[R], a: R, n: Nat, m: Nat) {
    ideal_quotient_mul(i,
        ideal_quotient_pow(i, ideal_quotient_project(i, a), n),
        ideal_quotient_pow(i, ideal_quotient_project(i, a), m)) =
    ideal_quotient_pow(i, ideal_quotient_project(i, a), n + m)
} by {
    ideal_quotient_pow_add(i, a, n, m)
}

/// Raising a projected quotient power to a power multiplies the exponents.
theorem ideal_quotient_project_pow_pow[R: CommRing](i: Ideal[R], a: R, n: Nat, m: Nat) {
    ideal_quotient_pow(i,
        ideal_quotient_pow(i, ideal_quotient_project(i, a), n),
        m) =
    ideal_quotient_pow(i, ideal_quotient_project(i, a), n * m)
} by {
    ideal_quotient_pow_pow(i, a, n, m)
}

/// The quotient power of a product is the product of the quotient powers.
theorem ideal_quotient_pow_mul[R: CommRing](i: Ideal[R], a: R, b: R, n: Nat) {
    ideal_quotient_pow(i,
        ideal_quotient_mul(i,
            quotient_over_mk(ideal_quotient_relation(i), a),
            quotient_over_mk(ideal_quotient_relation(i), b)),
        n) =
    ideal_quotient_mul(i,
        ideal_quotient_pow(i, quotient_over_mk(ideal_quotient_relation(i), a), n),
        ideal_quotient_pow(i, quotient_over_mk(ideal_quotient_relation(i), b), n))
} by {
    ideal_quotient_mul_projection(i, a, b)
    ideal_quotient_pow_mk(i, a * b, n)
    ideal_quotient_pow_mk(i, a, n)
    ideal_quotient_pow_mk(i, b, n)
    ideal_quotient_mul_projection(i, a.pow(n), b.pow(n))
    quotient_over_mk(ideal_quotient_relation(i), (a * b).pow(n)) =
        quotient_over_mk(ideal_quotient_relation(i), a.pow(n) * b.pow(n))
}

/// The projected quotient power of a product is the product of the projected quotient powers.
theorem ideal_quotient_project_pow_mul[R: CommRing](i: Ideal[R], a: R, b: R, n: Nat) {
    ideal_quotient_pow(i,
        ideal_quotient_mul(i, ideal_quotient_project(i, a), ideal_quotient_project(i, b)),
        n) =
    ideal_quotient_mul(i,
        ideal_quotient_pow(i, ideal_quotient_project(i, a), n),
        ideal_quotient_pow(i, ideal_quotient_project(i, b), n))
} by {
    ideal_quotient_pow_mul(i, a, b, n)
}

/// The square of a projected quotient representative is the projection of its square.
theorem ideal_quotient_pow_two[R: CommRing](i: Ideal[R], a: R) {
    ideal_quotient_pow(i,
        quotient_over_mk(ideal_quotient_relation(i), a), Nat.2) =
        quotient_over_mk(ideal_quotient_relation(i), a * a)
} by {
    ideal_quotient_pow_mk(i, a, Nat.2)
    a.pow(Nat.2) = a * a
}

/// The square of a projected quotient representative is the projection of its square.
theorem ideal_quotient_project_pow_two[R: CommRing](i: Ideal[R], a: R) {
    ideal_quotient_pow(i, ideal_quotient_project(i, a), Nat.2) =
        ideal_quotient_project(i, a * a)
} by {
    ideal_quotient_pow_two(i, a)
}

/// The quotient projection of a sum lies at quotient zero exactly when the sum lies in the ideal.
theorem ideal_quotient_project_add_eq_zero_iff_contains[R: CommRing](i: Ideal[R], a: R, b: R) {
    (ideal_quotient_add(i, ideal_quotient_project(i, a), ideal_quotient_project(i, b)) =
        ideal_quotient_zero(i)) = i.contains(a + b)
} by {
    ideal_quotient_project_add(i, a, b)
    ideal_quotient_project_eq_zero_iff_contains(i, a + b)
}

/// The quotient projection of a product lies at quotient zero exactly when the product lies in the ideal.
theorem ideal_quotient_project_mul_eq_zero_iff_contains[R: CommRing](i: Ideal[R], a: R, b: R) {
    (ideal_quotient_mul(i, ideal_quotient_project(i, a), ideal_quotient_project(i, b)) =
        ideal_quotient_zero(i)) = i.contains(a * b)
} by {
    ideal_quotient_project_mul(i, a, b)
    ideal_quotient_project_eq_zero_iff_contains(i, a * b)
}

/// The quotient projection of a power lies at quotient zero exactly when the power lies in the ideal.
theorem ideal_quotient_project_pow_eq_zero_iff_contains[R: CommRing](i: Ideal[R], a: R, n: Nat) {
    (ideal_quotient_pow(i, ideal_quotient_project(i, a), n) = ideal_quotient_zero(i)) =
        i.contains(a.pow(n))
} by {
    ideal_quotient_project_pow(i, a, n)
    ideal_quotient_project_eq_zero_iff_contains(i, a.pow(n))
}

/// Two quotient projections agree exactly when their sum lies in the ideal up to sign.
theorem ideal_quotient_project_eq_iff_contains_sub_swap[R: CommRing](i: Ideal[R], a: R, b: R) {
    (ideal_quotient_project(i, a) = ideal_quotient_project(i, b)) = i.contains(b - a)
} by {
    ideal_quotient_project_eq_iff_contains_sub(i, b, a)
    if ideal_quotient_project(i, a) = ideal_quotient_project(i, b) {
        i.contains(b - a)
    }
    if i.contains(b - a) {
        ideal_quotient_project(i, a) = ideal_quotient_project(i, b)
    }
}

/// The quotient projection equals quotient one exactly when the representative minus one lies in the ideal.
theorem ideal_quotient_project_eq_one_iff_contains_sub_one[R: CommRing](i: Ideal[R], a: R) {
    (ideal_quotient_project(i, a) = ideal_quotient_one(i)) = i.contains(a - R.1)
} by {
    ideal_quotient_project_one(i)
    ideal_quotient_project_eq_iff_contains_sub(i, a, R.1)
}

/// Quotient one equals the quotient projection exactly when one minus the representative lies in the ideal.
theorem ideal_quotient_one_eq_project_iff_contains_one_sub[R: CommRing](i: Ideal[R], a: R) {
    (ideal_quotient_one(i) = ideal_quotient_project(i, a)) = i.contains(R.1 - a)
} by {
    ideal_quotient_project_one(i)
    ideal_quotient_project_eq_iff_contains_sub(i, R.1, a)
}

/// The zeroth power of a projected representative is quotient one.
theorem ideal_quotient_project_pow_zero[R: CommRing](i: Ideal[R], a: R) {
    ideal_quotient_pow(i, ideal_quotient_project(i, a), Nat.0) =
        ideal_quotient_one(i)
} by {
    ideal_quotient_pow_zero(i, ideal_quotient_project(i, a))
}

/// The successor quotient power of a projected representative unfolds via multiplication.
theorem ideal_quotient_project_pow_suc[R: CommRing](i: Ideal[R], a: R, n: Nat) {
    ideal_quotient_pow(i, ideal_quotient_project(i, a), n.suc) =
        ideal_quotient_mul(i,
            ideal_quotient_project(i, a),
            ideal_quotient_pow(i, ideal_quotient_project(i, a), n))
} by {
    ideal_quotient_pow_suc(i, ideal_quotient_project(i, a), n)
}

/// The zeroth power of quotient zero is quotient one.
theorem ideal_quotient_zero_pow_zero[R: CommRing](i: Ideal[R]) {
    ideal_quotient_pow(i, ideal_quotient_zero(i), Nat.0) = ideal_quotient_one(i)
} by {
    ideal_quotient_pow_zero(i, ideal_quotient_zero(i))
}

/// A successor power of quotient zero is quotient zero.
theorem ideal_quotient_zero_pow_suc[R: CommRing](i: Ideal[R], n: Nat) {
    ideal_quotient_pow(i, ideal_quotient_zero(i), n.suc) = ideal_quotient_zero(i)
} by {
    ideal_quotient_zero_projection(i)
    ideal_quotient_pow_mk(i, R.0, n.suc)
    quotient_over_mk(ideal_quotient_relation(i), R.0.pow(n.suc)) =
        quotient_over_mk(ideal_quotient_relation(i), R.0)
}

/// The quotient projection of one is quotient one.
theorem ideal_quotient_project_pow_one_eq_one[R: CommRing](i: Ideal[R], n: Nat) {
    ideal_quotient_pow(i, ideal_quotient_project(i, R.1), n) = ideal_quotient_one(i)
} by {
    ideal_quotient_project_one(i)
    ideal_quotient_one_pow(i, n)
}
