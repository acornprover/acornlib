// Ideal-theory deepening: closure of ideals under subtraction and negation
// (as iffs), powers of ideal elements, containment of multiples, and the
// interaction of ideal membership with the intersection and sum of ideals,
// together with further coset identities in the quotient ring `R/I`.

from comm_ring import CommRing
from algebra.ring.ring import Ring
from algebra.ring.ideal import Ideal, ideal_contains_zero, ideal_contains_add, ideal_contains_neg,
    ideal_contains_mul_left, ideal_contains_mul_right, bundled_ideal_sum_contains,
    bundled_ideal_sum_contains_eq, bundled_ideal_inter_contains_eq
from algebra.ring.quotient_ring import IdealQuotient, ideal_quotient_mk_type, ideal_quotient_mk_type_add,
    ideal_quotient_mk_type_mul, ideal_quotient_mk_type_one, ideal_quotient_type_neg_mk,
    ideal_quotient_typeclass_neg_eq
from algebra.ring.ring_theorems_deep import ideal_quotient_mk_type_eq_iff_contains_sub
from algebra.ring.ring_axioms_deep import ring_sub_add_cancel, ring_sub_self, ring_sub_neg,
    ring_sub_eq_zero_iff_eq
from algebra.add_comm_group import sub_add_cancel
from algebra.one import One
from algebra.zero import Zero
from nat import Nat, pow_one, pow_add, pow_zero, alt_induction, from_nat
numerals Nat

/// An ideal is closed under subtraction.
theorem ideal_contains_sub[R: CommRing](i: Ideal[R], a: R, b: R) {
    i.contains(a) and i.contains(b) implies i.contains(a - b)
} by {
    if i.contains(a) and i.contains(b) {
        ideal_contains_neg(i, b)
        i.contains(-b)
        ideal_contains_add(i, a, -b)
        i.contains(a + -b)
        a - b = a + -b
        i.contains(a - b)
    }
}

/// Membership is preserved by negation, in both directions.
theorem ideal_contains_neg_iff[R: CommRing](i: Ideal[R], a: R) {
    i.contains(-a) = i.contains(a)
} by {
    if i.contains(-a) {
        ideal_contains_neg(i, -a)
        i.contains(--a)
        --a = a
        i.contains(a)
    }
    if i.contains(a) {
        ideal_contains_neg(i, a)
        i.contains(-a)
    }
    i.contains(-a) = i.contains(a)
}

/// A difference in the ideal and the subtrahend in the ideal put the minuend in the ideal.
theorem ideal_contains_of_contains_sub_add[R: CommRing](i: Ideal[R], a: R, b: R) {
    i.contains(a - b) and i.contains(b) implies i.contains(a)
} by {
    if i.contains(a - b) and i.contains(b) {
        ideal_contains_sub(i, a - b, -b)
        ideal_contains_neg(i, b)
        i.contains(-b)
        i.contains((a - b) - -b)
        (a - b) - -b = a - b + b
        ring_sub_neg(a - b, b)
        (a - b) - -b = a - b + b
        ring_sub_add_cancel(a, b)
        a - b + b = a
        (a - b) - -b = a
        i.contains(a)
    }
}

/// An element lies in the ideal exactly when its difference from zero does.
theorem ideal_contains_iff_contains_zero_sub[R: CommRing](i: Ideal[R], a: R) {
    i.contains(R.0 - a) = i.contains(a)
} by {
    R.0 - a = -a
    ideal_contains_neg_iff(i, a)
    i.contains(-a) = i.contains(a)
    i.contains(R.0 - a) = i.contains(a)
}

/// Every element of an ideal has a zero difference with itself in the ideal.
theorem ideal_contains_sub_self[R: CommRing](i: Ideal[R], a: R) {
    i.contains(a - a)
} by {
    ring_sub_self(a)
    a - a = R.0
    ideal_contains_zero(i)
    i.contains(R.0)
    i.contains(a - a)
}

/// An ideal element to any positive power stays in the ideal.
theorem ideal_contains_pow[R: CommRing](i: Ideal[R], a: R, n: Nat) {
    n != Nat.0 and i.contains(a) implies i.contains(a.pow(n))
} by {
    if n != Nat.0 and i.contains(a) {
        let k: Nat satisfy { n = k.suc }
        define p(m: Nat) -> Bool {
            i.contains(a.pow(m.suc))
        }
        pow_one(a)
        a.pow(Nat.1) = a
        i.contains(a.pow(Nat.1))
        p(Nat.0)
        forall(m: Nat) {
            if p(m) {
                pow_add(a, m.suc, Nat.1)
                a.pow(m.suc) * a.pow(Nat.1) = a.pow(m.suc + Nat.1)
                pow_one(a)
                a.pow(Nat.1) = a
                m.suc + Nat.1 = m.suc.suc
                a.pow(m.suc + Nat.1) = a.pow(m.suc.suc)
                a.pow(m.suc) * a = a.pow(m.suc.suc)
                i.contains(a.pow(m.suc))
                ideal_contains_mul_right(i, a.pow(m.suc), a)
                i.contains(a.pow(m.suc) * a)
                i.contains(a.pow(m.suc.suc))
                p(m.suc)
            }
        }
        p(Nat.0) and forall(m: Nat) { p(m) implies p(m.suc) }
        alt_induction(p)
        forall(m: Nat) { p(m) }
        p(k)
        i.contains(a.pow(k.suc))
        i.contains(a.pow(n))
    }
}

/// The square of an ideal element lies in the ideal.
theorem ideal_contains_pow_two[R: CommRing](i: Ideal[R], a: R) {
    i.contains(a) implies i.contains(a.pow(2))
} by {
    if i.contains(a) {
        ideal_contains_pow(i, a, Nat.2)
        i.contains(a.pow(2))
    }
}

/// A multiple of an ideal element by any ring element lies in the ideal.
theorem ideal_contains_mul_left_of_contains[R: CommRing](i: Ideal[R], r: R, a: R) {
    i.contains(a) implies i.contains(r * a)
} by {
    if i.contains(a) {
        ideal_contains_mul_left(i, r, a)
        i.contains(r * a)
    }
}

/// A multiple of an ideal element on the right lies in the ideal.
theorem ideal_contains_mul_right_of_contains[R: CommRing](i: Ideal[R], a: R, r: R) {
    i.contains(a) implies i.contains(a * r)
} by {
    if i.contains(a) {
        ideal_contains_mul_right(i, a, r)
        i.contains(a * r)
    }
}

/// The natural-number multiple of an ideal element lies in the ideal.
theorem ideal_contains_from_nat_mul[R: CommRing](i: Ideal[R], a: R, n: Nat) {
    i.contains(a) implies i.contains(from_nat[R](n) * a)
} by {
    if i.contains(a) {
        ideal_contains_mul_left(i, from_nat[R](n), a)
        i.contains(from_nat[R](n) * a)
    }
}

/// The product of an element of `i` and an element of `j` lies in the intersection.
theorem ideal_inter_contains_mul[R: CommRing](i: Ideal[R], j: Ideal[R], a: R, b: R) {
    i.contains(a) and j.contains(b) implies i.intersection(j).contains(a * b)
} by {
    if i.contains(a) and j.contains(b) {
        ideal_contains_mul_right(i, a, b)
        i.contains(a * b)
        ideal_contains_mul_left(j, a, b)
        j.contains(a * b)
        bundled_ideal_inter_contains_eq(i, j, a * b)
        i.intersection(j).contains(a * b)
    }
}

/// The product of an element of `i` and an element of `j` lies in the sum.
theorem ideal_sum_contains_mul[R: CommRing](i: Ideal[R], j: Ideal[R], a: R, b: R) {
    i.contains(a) and j.contains(b) implies i.sum(j).contains(a * b)
} by {
    if i.contains(a) and j.contains(b) {
        ideal_contains_mul_right(i, a, b)
        i.contains(a * b)
        ideal_contains_zero(j)
        j.contains(R.0)
        bundled_ideal_sum_contains(i, j, a * b)
        bundled_ideal_sum_contains_eq(i, j, a * b)
        i.sum(j).contains(a * b)
    }
}

/// An element of `i` lies in the sum of `i` and `j`.
theorem ideal_sum_contains_left_elem[R: CommRing](i: Ideal[R], j: Ideal[R], a: R) {
    i.contains(a) implies i.sum(j).contains(a)
} by {
    if i.contains(a) {
        ideal_contains_zero(j)
        j.contains(R.0)
        bundled_ideal_sum_contains(i, j, a)
        bundled_ideal_sum_contains_eq(i, j, a)
        i.sum(j).contains(a)
    }
}

/// An element of `j` lies in the sum of `i` and `j`.
theorem ideal_sum_contains_right_elem[R: CommRing](i: Ideal[R], j: Ideal[R], b: R) {
    j.contains(b) implies i.sum(j).contains(b)
} by {
    if j.contains(b) {
        ideal_contains_zero(i)
        i.contains(R.0)
        bundled_ideal_sum_contains(i, j, b)
        bundled_ideal_sum_contains_eq(i, j, b)
        i.sum(j).contains(b)
    }
}

/// The canonical projection preserves subtraction.
theorem ideal_quotient_mk_type_sub[R: CommRing](i: Ideal[R], a: R, b: R) {
    ideal_quotient_mk_type(i, a - b) = ideal_quotient_mk_type(i, a) - ideal_quotient_mk_type(i, b)
} by {
    a - b = a + -b
    ideal_quotient_mk_type_add(i, a, -b)
    ideal_quotient_mk_type(i, a + -b) = ideal_quotient_mk_type(i, a) + ideal_quotient_mk_type(i, -b)
    ideal_quotient_type_neg_mk(i, b)
    ideal_quotient_typeclass_neg_eq(i, ideal_quotient_mk_type(i, b))
    -ideal_quotient_mk_type(i, b) = ideal_quotient_mk_type(i, -b)
    ideal_quotient_mk_type(i, a) + ideal_quotient_mk_type(i, -b) =
        ideal_quotient_mk_type(i, a) + -ideal_quotient_mk_type(i, b)
    ideal_quotient_mk_type(i, a) - ideal_quotient_mk_type(i, b) =
        ideal_quotient_mk_type(i, a) + -ideal_quotient_mk_type(i, b)
    ideal_quotient_mk_type(i, a - b) = ideal_quotient_mk_type(i, a) - ideal_quotient_mk_type(i, b)
}

/// The canonical projection preserves natural powers.
theorem ideal_quotient_mk_type_pow[R: CommRing](i: Ideal[R], a: R, n: Nat) {
    ideal_quotient_mk_type(i, a.pow(n)) = ideal_quotient_mk_type(i, a).pow(n)
} by {
    define p(m: Nat) -> Bool {
        ideal_quotient_mk_type(i, a.pow(m)) = ideal_quotient_mk_type(i, a).pow(m)
    }
    pow_zero(a)
    a.pow(Nat.0) = R.1
    ideal_quotient_mk_type(i, a.pow(Nat.0)) = ideal_quotient_mk_type(i, R.1)
    ideal_quotient_mk_type_one(i)
    pow_zero(ideal_quotient_mk_type(i, a))
    ideal_quotient_mk_type(i, a.pow(Nat.0)) = ideal_quotient_mk_type(i, a).pow(Nat.0)
    p(Nat.0)
    forall(m: Nat) {
        if p(m) {
            pow_add(a, m, Nat.1)
            a.pow(m) * a.pow(Nat.1) = a.pow(m + Nat.1)
            pow_one(a)
            a.pow(Nat.1) = a
            m + Nat.1 = m.suc
            a.pow(m + Nat.1) = a.pow(m.suc)
            a.pow(m) * a = a.pow(m.suc)
            ideal_quotient_mk_type_mul(i, a.pow(m), a)
            ideal_quotient_mk_type(i, a.pow(m) * a) =
                ideal_quotient_mk_type(i, a.pow(m)) * ideal_quotient_mk_type(i, a)
            p(m) = (ideal_quotient_mk_type(i, a.pow(m)) = ideal_quotient_mk_type(i, a).pow(m))
            ideal_quotient_mk_type(i, a.pow(m)) = ideal_quotient_mk_type(i, a).pow(m)
            pow_add(ideal_quotient_mk_type(i, a), m, Nat.1)
            ideal_quotient_mk_type(i, a).pow(m) * ideal_quotient_mk_type(i, a).pow(Nat.1) =
                ideal_quotient_mk_type(i, a).pow(m + Nat.1)
            pow_one(ideal_quotient_mk_type(i, a))
            ideal_quotient_mk_type(i, a).pow(Nat.1) = ideal_quotient_mk_type(i, a)
            ideal_quotient_mk_type(i, a).pow(m) * ideal_quotient_mk_type(i, a) =
                ideal_quotient_mk_type(i, a).pow(m.suc)
            ideal_quotient_mk_type(i, a.pow(m.suc)) = ideal_quotient_mk_type(i, a).pow(m.suc)
            p(m.suc)
        }
    }
    p(Nat.0) and forall(m: Nat) { p(m) implies p(m.suc) }
    alt_induction(p)
    forall(m: Nat) { p(m) }
    p(n)
}

/// The projection of a difference is zero exactly when the difference lies in the ideal.
theorem ideal_quotient_mk_type_sub_eq_zero_iff_contains[R: CommRing](i: Ideal[R], a: R, b: R) {
    (ideal_quotient_mk_type(i, a) - ideal_quotient_mk_type(i, b) = Zero.0[IdealQuotient[R, i]]) =
        i.contains(a - b)
} by {
    ring_sub_eq_zero_iff_eq(ideal_quotient_mk_type(i, a), ideal_quotient_mk_type(i, b))
    (ideal_quotient_mk_type(i, a) - ideal_quotient_mk_type(i, b) = Zero.0[IdealQuotient[R, i]]) =
        (ideal_quotient_mk_type(i, a) = ideal_quotient_mk_type(i, b))
    ideal_quotient_mk_type_eq_iff_contains_sub(i, a, b)
    (ideal_quotient_mk_type(i, a) = ideal_quotient_mk_type(i, b)) = i.contains(a - b)
    (ideal_quotient_mk_type(i, a) - ideal_quotient_mk_type(i, b) = Zero.0[IdealQuotient[R, i]]) =
        i.contains(a - b)
}
