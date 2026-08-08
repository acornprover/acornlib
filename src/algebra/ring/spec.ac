/// The prime spectrum `Spec(R)` of a commutative ring, whose points are the
/// prime ideals of `R`. The Zariski topology on this set is introduced
/// separately.

from comm_ring import CommRing
from algebra.ring.ideal import PrimeIdeal, prime_ideal_ext, Ideal, MaximalIdeal,
    bundled_ideal_subset, bundled_ideal_subset_trans,
    bundled_zero_ideal_subset, bundled_ideal_sum_subset_iff,
    prime_ideal_not_contains_one, prime_ideal_as_ideal_contains_eq,
    prime_ideal_contains_mul,
    bundled_unit_ideal_contains_everything,
    bundled_principal_ideal_subset_iff_contains_generator,
    bundled_ideal_inter_subset_left, bundled_ideal_inter_subset_right,
    bundled_ideal_inter_contains_eq,
    ideal_contains_mul_left, ideal_contains_mul_right,
    ideal_indexed_inter, ideal_indexed_inter_subset
from data.basic.set import Set, universal_set_compl_is_empty, empty_set_compl_is_universal,
    indexed_union, indexed_union_contains_witness,
    intersection_contains_eq, compl_of_compl_is_self, compl_contains_eq,
    union_contains_eq

/// A point of the prime spectrum `Spec(R)`: a prime ideal of `R`, packaged as
/// a distinct type so that the Zariski topology and structure sheaf can be
/// attached without conflating points with their underlying ideals.
structure Spec[R: CommRing] {
    /// The prime ideal of `R` that this point of the spectrum represents.
    point: PrimeIdeal[R]
}

/// Two points of `Spec(R)` are equal exactly when their underlying prime ideals are equal.
theorem spec_ext[R: CommRing](a: Spec[R], b: Spec[R]) {
    a.point = b.point implies a = b
}

/// Two points of `Spec(R)` are equal exactly when their membership predicates agree.
theorem spec_ext_contains[R: CommRing](a: Spec[R], b: Spec[R]) {
    (forall(x: R) { a.point.contains(x) = b.point.contains(x) }) implies a = b
} by {
    if forall(x: R) { a.point.contains(x) = b.point.contains(x) } {
        prime_ideal_ext(a.point, b.point)
    }
}

/// The point of `Spec(R)` corresponding to a prime ideal.
define spec_of_prime[R: CommRing](p: PrimeIdeal[R]) -> Spec[R] {
    Spec[R].new(p)
}

/// Recovering the prime ideal from `spec_of_prime`.
theorem spec_of_prime_point[R: CommRing](p: PrimeIdeal[R]) {
    spec_of_prime(p).point = p
}

attributes Spec[R: CommRing] {
    /// Extensionality: equality of points reduces to equality of prime ideals.
    let ext = spec_ext[R]

    /// Extensionality via membership predicates.
    let ext_contains = spec_ext_contains[R]

    /// The point of `Spec(R)` corresponding to a prime ideal of `R`.
    let of_prime: PrimeIdeal[R] -> Spec[R] = spec_of_prime[R]
}

/// Two points of `Spec(R)` agree as prime ideals exactly when their `of_prime` constructions agree.
theorem spec_of_prime_eq[R: CommRing](p: PrimeIdeal[R], q: PrimeIdeal[R]) {
    Spec[R].of_prime(p) = Spec[R].of_prime(q) implies p = q
} by {
    if Spec[R].of_prime(p) = Spec[R].of_prime(q) {
        Spec[R].of_prime(p).point = Spec[R].of_prime(q).point
        spec_of_prime_point(p)
        spec_of_prime_point(q)
    }
}

/// The point of `Spec(R)` corresponding to a maximal ideal of `R`. Every
/// maximal ideal is in particular a prime ideal, hence a closed point of the
/// spectrum.
define spec_of_maximal[R: CommRing](m: MaximalIdeal[R]) -> Spec[R] {
    Spec[R].of_prime(m.as_prime_ideal)
}

/// True if a point of `Spec(R)` belongs to the vanishing set `V(I)`, that is,
/// the underlying prime ideal contains `I`.
define spec_in_vanishing[R: CommRing](i: Ideal[R], p: Spec[R]) -> Bool {
    bundled_ideal_subset(i, p.point.as_ideal)
}

/// The vanishing set `V(I)`: the set of points of `Spec(R)` whose underlying
/// prime ideal contains the ideal `I`.
define spec_vanishing[R: CommRing](i: Ideal[R]) -> Set[Spec[R]] {
    Set[Spec[R]].new(function(p: Spec[R]) {
        spec_in_vanishing(i, p)
    })
}

attributes Spec[R: CommRing] {
    /// The point of `Spec(R)` corresponding to a maximal ideal of `R`.
    let of_maximal: MaximalIdeal[R] -> Spec[R] = spec_of_maximal[R]

    /// The vanishing set `V(I)` of an ideal of `R`.
    let vanishing: Ideal[R] -> Set[Spec[R]] = spec_vanishing[R]
}

/// Membership in `V(I)` is exactly containment of `I` in the underlying prime ideal.
theorem spec_vanishing_contains_eq[R: CommRing](i: Ideal[R], p: Spec[R]) {
    Spec[R].vanishing(i).contains(p) = bundled_ideal_subset(i, p.point.as_ideal)
}

/// `V` is inclusion-reversing: a larger ideal cuts out a smaller vanishing set.
theorem spec_vanishing_antitone[R: CommRing](i: Ideal[R], j: Ideal[R]) {
    bundled_ideal_subset(i, j) implies Spec[R].vanishing(j).subset(Spec[R].vanishing(i))
} by {
    if bundled_ideal_subset(i, j) {
        forall(p: Spec[R]) {
            if Spec[R].vanishing(j).contains(p) {
                spec_vanishing_contains_eq(j, p)
                bundled_ideal_subset(j, p.point.as_ideal)
                bundled_ideal_subset_trans(i, j, p.point.as_ideal)
                bundled_ideal_subset(i, p.point.as_ideal)
                spec_vanishing_contains_eq(i, p)
                Spec[R].vanishing(i).contains(p)
            }
        }
    }
}

/// The vanishing set of the zero ideal is the entire spectrum.
theorem spec_vanishing_zero[R: CommRing] {
    Spec[R].vanishing(Ideal[R].zero) = Set[Spec[R]].universal_set
} by {
    forall(p: Spec[R]) {
        bundled_zero_ideal_subset(p.point.as_ideal)
        spec_vanishing_contains_eq(Ideal[R].zero, p)
        Spec[R].vanishing(Ideal[R].zero).contains(p)
    }
    forall(p: Spec[R]) {
        Spec[R].vanishing(Ideal[R].zero).contains(p)
    }
}

/// The vanishing set of the unit ideal is empty: no prime ideal contains `1`.
theorem spec_vanishing_unit[R: CommRing] {
    Spec[R].vanishing(Ideal[R].unit) = Set[Spec[R]].empty_set
} by {
    forall(p: Spec[R]) {
        if Spec[R].vanishing(Ideal[R].unit).contains(p) {
            spec_vanishing_contains_eq(Ideal[R].unit, p)
            bundled_ideal_subset(Ideal[R].unit, p.point.as_ideal)
            bundled_unit_ideal_contains_everything[R](R.1)
            Ideal[R].unit.contains(R.1)
            p.point.as_ideal.contains(R.1)
            prime_ideal_as_ideal_contains_eq(p.point, R.1)
            p.point.contains(R.1)
            prime_ideal_not_contains_one(p.point)
            false
        }
    }
}

/// Forward direction of `V(I + J) = V(I) ∩ V(J)`.
theorem spec_vanishing_sum_subset_inter[R: CommRing](i: Ideal[R], j: Ideal[R]) {
    Spec[R].vanishing(i.sum(j)).subset(Spec[R].vanishing(i).intersection(Spec[R].vanishing(j)))
} by {
    forall(p: Spec[R]) {
        if Spec[R].vanishing(i.sum(j)).contains(p) {
            spec_vanishing_contains_eq(i.sum(j), p)
            bundled_ideal_sum_subset_iff(i, j, p.point.as_ideal)
            spec_vanishing_contains_eq(i, p)
            spec_vanishing_contains_eq(j, p)
            Spec[R].vanishing(i).contains(p)
            Spec[R].vanishing(j).contains(p)
            Spec[R].vanishing(i).intersection(Spec[R].vanishing(j)).contains(p)
        }
    }
}

/// Backward direction of `V(I + J) = V(I) ∩ V(J)`.
theorem spec_vanishing_inter_subset_sum[R: CommRing](i: Ideal[R], j: Ideal[R]) {
    Spec[R].vanishing(i).intersection(Spec[R].vanishing(j)).subset(Spec[R].vanishing(i.sum(j)))
} by {
    forall(p: Spec[R]) {
        if Spec[R].vanishing(i).intersection(Spec[R].vanishing(j)).contains(p) {
            Spec[R].vanishing(i).contains(p)
            Spec[R].vanishing(j).contains(p)
            spec_vanishing_contains_eq(i, p)
            spec_vanishing_contains_eq(j, p)
            bundled_ideal_sum_subset_iff(i, j, p.point.as_ideal)
            spec_vanishing_contains_eq(i.sum(j), p)
            Spec[R].vanishing(i.sum(j)).contains(p)
        }
    }
}

/// `V(I + J) = V(I) ∩ V(J)`: a prime contains the sum exactly when it contains both summands.
theorem spec_vanishing_sum[R: CommRing](i: Ideal[R], j: Ideal[R]) {
    Spec[R].vanishing(i.sum(j)) = Spec[R].vanishing(i).intersection(Spec[R].vanishing(j))
} by {
    spec_vanishing_sum_subset_inter(i, j)
    spec_vanishing_inter_subset_sum(i, j)
}

/// Recovering the prime ideal from `Spec.of_maximal`.
theorem spec_of_maximal_point[R: CommRing](m: MaximalIdeal[R]) {
    Spec[R].of_maximal(m).point = m.as_prime_ideal
} by {
    Spec[R].of_maximal(m) = Spec[R].of_prime(m.as_prime_ideal)
    spec_of_prime_point(m.as_prime_ideal)
}

/// Membership in `V(principal(f))` is exactly containment of `f` in the
/// underlying prime ideal.
theorem spec_vanishing_principal_contains_eq[R: CommRing](f: R, p: Spec[R]) {
    Spec[R].vanishing(Ideal[R].principal(f)).contains(p) = p.point.contains(f)
} by {
    spec_vanishing_contains_eq(Ideal[R].principal(f), p)
    bundled_principal_ideal_subset_iff_contains_generator(f, p.point.as_ideal)
    prime_ideal_as_ideal_contains_eq(p.point, f)
}

/// The vanishing set of the principal ideal generated by `0` is the entire spectrum.
theorem spec_vanishing_principal_zero[R: CommRing] {
    Spec[R].vanishing(Ideal[R].principal(R.0)) = Set[Spec[R]].universal_set
} by {
    forall(p: Spec[R]) {
        p.point.contains(R.0)
        spec_vanishing_principal_contains_eq(R.0, p)
        Spec[R].vanishing(Ideal[R].principal(R.0)).contains(p)
    }
}

/// The vanishing set of the principal ideal generated by `1` is empty: no
/// prime ideal contains `1`.
theorem spec_vanishing_principal_one[R: CommRing] {
    Spec[R].vanishing(Ideal[R].principal(R.1)) = Set[Spec[R]].empty_set
} by {
    forall(p: Spec[R]) {
        if Spec[R].vanishing(Ideal[R].principal(R.1)).contains(p) {
            spec_vanishing_principal_contains_eq(R.1, p)
            p.point.contains(R.1)
            prime_ideal_not_contains_one(p.point)
            false
        }
    }
}

/// The distinguished open set `D(f)`: the complement of `V(principal(f))`,
/// that is, the set of points where `f` does not vanish.
define spec_basic_open[R: CommRing](f: R) -> Set[Spec[R]] {
    Spec[R].vanishing(Ideal[R].principal(f)).c
}

attributes Spec[R: CommRing] {
    /// The distinguished open set `D(f)` complementary to `V(principal(f))`.
    let basic_open: R -> Set[Spec[R]] = spec_basic_open[R]
}

/// Membership in the distinguished open set `D(f)` is exactly non-containment
/// of `f` in the underlying prime ideal.
theorem spec_basic_open_contains_eq[R: CommRing](f: R, p: Spec[R]) {
    Spec[R].basic_open(f).contains(p) = not p.point.contains(f)
} by {
    Spec[R].basic_open(f) = Spec[R].vanishing(Ideal[R].principal(f)).c
    Spec[R].basic_open(f).contains(p) = not Spec[R].vanishing(Ideal[R].principal(f)).contains(p)
    spec_vanishing_principal_contains_eq(f, p)
}

/// `D(0) = ∅`: the basic open set at zero is empty.
theorem spec_basic_open_zero[R: CommRing] {
    Spec[R].basic_open(R.0) = Set[Spec[R]].empty_set
} by {
    Spec[R].basic_open(R.0) = Spec[R].vanishing(Ideal[R].principal(R.0)).c
    spec_vanishing_principal_zero[R]
    Spec[R].basic_open(R.0) = (Set[Spec[R]].universal_set).c
    universal_set_compl_is_empty[Spec[R]]
}

/// `D(1) = Spec(R)`: the basic open set at one is the whole spectrum.
theorem spec_basic_open_one[R: CommRing] {
    Spec[R].basic_open(R.1) = Set[Spec[R]].universal_set
} by {
    Spec[R].basic_open(R.1) = Spec[R].vanishing(Ideal[R].principal(R.1)).c
    spec_vanishing_principal_one[R]
    Spec[R].basic_open(R.1) = (Set[Spec[R]].empty_set).c
    empty_set_compl_is_universal[Spec[R]]
}

/// `V(I) ∪ V(J) ⊆ V(I ∩ J)`: any prime containing either ideal contains
/// their intersection. (The reverse inclusion fails in general and is the
/// content of the lemma `V(I ∩ J) = V(I·J)`, deferred until ideal products
/// are available.)
theorem spec_vanishing_union_subset_inter[R: CommRing](i: Ideal[R], j: Ideal[R]) {
    Spec[R].vanishing(i).union(Spec[R].vanishing(j)).subset(Spec[R].vanishing(i.intersection(j)))
} by {
    bundled_ideal_inter_subset_left(i, j)
    bundled_ideal_inter_subset_right(i, j)
    spec_vanishing_antitone(i.intersection(j), i)
    spec_vanishing_antitone(i.intersection(j), j)
    forall(p: Spec[R]) {
        if Spec[R].vanishing(i).union(Spec[R].vanishing(j)).contains(p) {
            if Spec[R].vanishing(i).contains(p) {
                Spec[R].vanishing(i.intersection(j)).contains(p)
            }
            if Spec[R].vanishing(j).contains(p) {
                Spec[R].vanishing(i.intersection(j)).contains(p)
            }
            Spec[R].vanishing(i.intersection(j)).contains(p)
        }
    }
}

/// `V(I ∩ J) ⊆ V(I) ∪ V(J)`: a prime containing the intersection of two
/// ideals contains at least one of them. Together with `V(I) ∪ V(J) ⊆ V(I ∩ J)`
/// this gives the equality `V(I) ∪ V(J) = V(I ∩ J)`.
theorem spec_vanishing_inter_subset_union[R: CommRing](i: Ideal[R], j: Ideal[R]) {
    Spec[R].vanishing(i.intersection(j)).subset(
        Spec[R].vanishing(i).union(Spec[R].vanishing(j)))
} by {
    forall(p: Spec[R]) {
        if Spec[R].vanishing(i.intersection(j)).contains(p) {
            spec_vanishing_contains_eq(i.intersection(j), p)
            bundled_ideal_subset(i.intersection(j), p.point.as_ideal)
            if not bundled_ideal_subset(i, p.point.as_ideal) {
                if not bundled_ideal_subset(j, p.point.as_ideal) {
                    let a: R satisfy {
                        i.contains(a) and not p.point.as_ideal.contains(a)
                    }
                    let b: R satisfy {
                        j.contains(b) and not p.point.as_ideal.contains(b)
                    }
                    ideal_contains_mul_right(i, a, b)
                    i.contains(a * b)
                    ideal_contains_mul_left(j, a, b)
                    j.contains(a * b)
                    bundled_ideal_inter_contains_eq(i, j, a * b)
                    i.intersection(j).contains(a * b)
                    p.point.as_ideal.contains(a * b)
                    prime_ideal_as_ideal_contains_eq(p.point, a * b)
                    p.point.contains(a * b)
                    prime_ideal_contains_mul(p.point, a, b)
                    p.point.contains(a) or p.point.contains(b)
                    prime_ideal_as_ideal_contains_eq(p.point, a)
                    prime_ideal_as_ideal_contains_eq(p.point, b)
                    false
                }
                bundled_ideal_subset(j, p.point.as_ideal)
                spec_vanishing_contains_eq(j, p)
                Spec[R].vanishing(j).contains(p)
                Spec[R].vanishing(i).union(Spec[R].vanishing(j)).contains(p)
            }
            if bundled_ideal_subset(i, p.point.as_ideal) {
                spec_vanishing_contains_eq(i, p)
                Spec[R].vanishing(i).contains(p)
                Spec[R].vanishing(i).union(Spec[R].vanishing(j)).contains(p)
            }
            Spec[R].vanishing(i).union(Spec[R].vanishing(j)).contains(p)
        }
    }
}

/// `V(I) ∪ V(J) = V(I ∩ J)`: a prime contains the intersection of two ideals
/// exactly when it contains one of them.
theorem spec_vanishing_union_eq_inter[R: CommRing](i: Ideal[R], j: Ideal[R]) {
    Spec[R].vanishing(i).union(Spec[R].vanishing(j)) =
        Spec[R].vanishing(i.intersection(j))
} by {
    spec_vanishing_union_subset_inter(i, j)
    spec_vanishing_inter_subset_union(i, j)
}

/// The complement of `D(f)` is the vanishing set of the principal ideal `(f)`.
theorem spec_basic_open_compl[R: CommRing](f: R) {
    Spec[R].basic_open(f).c = Spec[R].vanishing(Ideal[R].principal(f))
} by {
    Spec[R].basic_open(f) = Spec[R].vanishing(Ideal[R].principal(f)).c
    compl_of_compl_is_self(Spec[R].vanishing(Ideal[R].principal(f)))
}

/// The complement of `V(principal(f))` is the distinguished open set `D(f)`.
theorem spec_vanishing_principal_compl[R: CommRing](f: R) {
    Spec[R].vanishing(Ideal[R].principal(f)).c = Spec[R].basic_open(f)
} by {
    Spec[R].basic_open(f) = Spec[R].vanishing(Ideal[R].principal(f)).c
}

/// The intersection of two basic-open complements `(V(I))^c ∩ (V(J))^c`
/// is the complement of `V(I ∩ J)`.
theorem spec_vanishing_compl_inter[R: CommRing](i: Ideal[R], j: Ideal[R]) {
    Spec[R].vanishing(i).c.intersection(Spec[R].vanishing(j).c) =
        Spec[R].vanishing(i.intersection(j)).c
} by {
    spec_vanishing_union_eq_inter(i, j)
    forall(p: Spec[R]) {
        intersection_contains_eq(Spec[R].vanishing(i).c, Spec[R].vanishing(j).c, p)
        compl_contains_eq(Spec[R].vanishing(i), p)
        compl_contains_eq(Spec[R].vanishing(j), p)
        union_contains_eq(Spec[R].vanishing(i), Spec[R].vanishing(j), p)
        compl_contains_eq(Spec[R].vanishing(i.intersection(j)), p)
        if Spec[R].vanishing(i).c.intersection(Spec[R].vanishing(j).c).contains(p) {
            not Spec[R].vanishing(i).contains(p)
            not Spec[R].vanishing(j).contains(p)
            not Spec[R].vanishing(i).union(Spec[R].vanishing(j)).contains(p)
            not Spec[R].vanishing(i.intersection(j)).contains(p)
            Spec[R].vanishing(i.intersection(j)).c.contains(p)
        }
        if Spec[R].vanishing(i.intersection(j)).c.contains(p) {
            not Spec[R].vanishing(i.intersection(j)).contains(p)
            not Spec[R].vanishing(i).union(Spec[R].vanishing(j)).contains(p)
            not Spec[R].vanishing(i).contains(p)
            not Spec[R].vanishing(j).contains(p)
            Spec[R].vanishing(i).c.contains(p)
            Spec[R].vanishing(j).c.contains(p)
            Spec[R].vanishing(i).c.intersection(Spec[R].vanishing(j).c).contains(p)
        }
        Spec[R].vanishing(i).c.intersection(Spec[R].vanishing(j).c).contains(p) =
            Spec[R].vanishing(i.intersection(j)).c.contains(p)
    }
}

/// `Spec.of_prime` is injective: distinct prime ideals give distinct points.
theorem spec_of_prime_inj[R: CommRing](p: PrimeIdeal[R], q: PrimeIdeal[R]) {
    Spec[R].of_prime(p) = Spec[R].of_prime(q) implies p = q
} by {
    if Spec[R].of_prime(p) = Spec[R].of_prime(q) {
        spec_of_prime_eq(p, q)
    }
}

/// The family of vanishing sets attached to an indexed family of ideals.
define spec_vanishing_family[R: CommRing, I](family: I -> Ideal[R], i: I) -> Set[Spec[R]] {
    Spec[R].vanishing(family(i))
}

/// `⋃_i V(I_i) ⊆ V(⋂_i I_i)`: any prime containing some member of an indexed
/// family of ideals also contains their intersection.
theorem spec_vanishing_indexed_union_subset_inter[R: CommRing, I](family: I -> Ideal[R]) {
    indexed_union(spec_vanishing_family(family)).subset(
        Spec[R].vanishing(ideal_indexed_inter(family)))
} by {
    forall(p: Spec[R]) {
        if indexed_union(spec_vanishing_family(family)).contains(p) {
            indexed_union_contains_witness(spec_vanishing_family(family), p)
            exists(i: I) {
                spec_vanishing_family(family)(i).contains(p)
            }
            let i: I satisfy { spec_vanishing_family(family)(i).contains(p) }
            spec_vanishing_family(family)(i) = Spec[R].vanishing(family(i))
            Spec[R].vanishing(family(i)).contains(p)
            ideal_indexed_inter_subset(family, i)
            spec_vanishing_antitone(ideal_indexed_inter(family), family(i))
            Spec[R].vanishing(family(i)).subset(Spec[R].vanishing(ideal_indexed_inter(family)))
            Spec[R].vanishing(ideal_indexed_inter(family)).contains(p)
        }
    }
}

/// `D(f * g) = D(f) ∩ D(g)`: a prime fails to contain `f * g` exactly when it
/// fails to contain both `f` and `g`.
theorem spec_basic_open_mul[R: CommRing](f: R, g: R) {
    Spec[R].basic_open(f * g) = Spec[R].basic_open(f).intersection(Spec[R].basic_open(g))
} by {
    forall(p: Spec[R]) {
        spec_basic_open_contains_eq(f * g, p)
        spec_basic_open_contains_eq(f, p)
        spec_basic_open_contains_eq(g, p)
        prime_ideal_contains_mul(p.point, f, g)
        if p.point.contains(f) {
            prime_ideal_as_ideal_contains_eq(p.point, f)
            p.point.as_ideal.contains(f)
            ideal_contains_mul_right(p.point.as_ideal, f, g)
            p.point.as_ideal.contains(f * g)
            prime_ideal_as_ideal_contains_eq(p.point, f * g)
            p.point.contains(f * g)
        }
        if p.point.contains(g) {
            prime_ideal_as_ideal_contains_eq(p.point, g)
            p.point.as_ideal.contains(g)
            ideal_contains_mul_left(p.point.as_ideal, f, g)
            p.point.as_ideal.contains(f * g)
            prime_ideal_as_ideal_contains_eq(p.point, f * g)
            p.point.contains(f * g)
        }
        p.point.contains(f * g) = (p.point.contains(f) or p.point.contains(g))
        intersection_contains_eq(Spec[R].basic_open(f), Spec[R].basic_open(g), p)
        if Spec[R].basic_open(f * g).contains(p) {
            not p.point.contains(f * g)
            not p.point.contains(f)
            not p.point.contains(g)
            Spec[R].basic_open(f).contains(p)
            Spec[R].basic_open(g).contains(p)
            Spec[R].basic_open(f).intersection(Spec[R].basic_open(g)).contains(p)
        }
        if Spec[R].basic_open(f).intersection(Spec[R].basic_open(g)).contains(p) {
            Spec[R].basic_open(f).contains(p)
            Spec[R].basic_open(g).contains(p)
            not p.point.contains(f)
            not p.point.contains(g)
            not p.point.contains(f * g)
            Spec[R].basic_open(f * g).contains(p)
        }
        Spec[R].basic_open(f * g).contains(p) =
            Spec[R].basic_open(f).intersection(Spec[R].basic_open(g)).contains(p)
    }
}

/// `Spec.of_prime(p) ∈ V(I)` exactly when `I` is contained in the prime `p`.
theorem spec_of_prime_in_vanishing[R: CommRing](i: Ideal[R], p: PrimeIdeal[R]) {
    Spec[R].vanishing(i).contains(Spec[R].of_prime(p)) = bundled_ideal_subset(i, p.as_ideal)
} by {
    spec_vanishing_contains_eq(i, Spec[R].of_prime(p))
    spec_of_prime_point(p)
}

/// `Spec.of_maximal(m) ∈ V(I)` exactly when `I` is contained in the maximal ideal `m`.
theorem spec_of_maximal_in_vanishing[R: CommRing](i: Ideal[R], m: MaximalIdeal[R]) {
    Spec[R].vanishing(i).contains(Spec[R].of_maximal(m)) =
        bundled_ideal_subset(i, m.as_prime_ideal.as_ideal)
} by {
    spec_of_maximal_point(m)
    spec_vanishing_contains_eq(i, Spec[R].of_maximal(m))
}

/// The complement of `V(0)` is empty.
theorem spec_vanishing_zero_compl[R: CommRing] {
    Spec[R].vanishing(Ideal[R].zero).c = Set[Spec[R]].empty_set
} by {
    spec_vanishing_zero[R]
    universal_set_compl_is_empty[Spec[R]]
}

/// The complement of `V(unit)` is the whole spectrum.
theorem spec_vanishing_unit_compl[R: CommRing] {
    Spec[R].vanishing(Ideal[R].unit).c = Set[Spec[R]].universal_set
} by {
    spec_vanishing_unit[R]
    empty_set_compl_is_universal[Spec[R]]
}

/// `V(I + J).c = V(I).c ∪ V(J).c`: the complement of a sum vanishing set is the
/// union of complements.
theorem spec_vanishing_sum_compl[R: CommRing](i: Ideal[R], j: Ideal[R]) {
    Spec[R].vanishing(i.sum(j)).c =
        Spec[R].vanishing(i).c.union(Spec[R].vanishing(j).c)
} by {
    spec_vanishing_sum(i, j)
    forall(p: Spec[R]) {
        intersection_contains_eq(Spec[R].vanishing(i), Spec[R].vanishing(j), p)
        compl_contains_eq(Spec[R].vanishing(i.sum(j)), p)
        compl_contains_eq(Spec[R].vanishing(i), p)
        compl_contains_eq(Spec[R].vanishing(j), p)
        union_contains_eq(Spec[R].vanishing(i).c, Spec[R].vanishing(j).c, p)
        if Spec[R].vanishing(i.sum(j)).c.contains(p) {
            not Spec[R].vanishing(i.sum(j)).contains(p)
            not Spec[R].vanishing(i).intersection(Spec[R].vanishing(j)).contains(p)
            not (Spec[R].vanishing(i).contains(p) and Spec[R].vanishing(j).contains(p))
            not Spec[R].vanishing(i).contains(p) or not Spec[R].vanishing(j).contains(p)
            Spec[R].vanishing(i).c.contains(p) or Spec[R].vanishing(j).c.contains(p)
            Spec[R].vanishing(i).c.union(Spec[R].vanishing(j).c).contains(p)
        }
        if Spec[R].vanishing(i).c.union(Spec[R].vanishing(j).c).contains(p) {
            Spec[R].vanishing(i).c.contains(p) or Spec[R].vanishing(j).c.contains(p)
            not Spec[R].vanishing(i).contains(p) or not Spec[R].vanishing(j).contains(p)
            not (Spec[R].vanishing(i).contains(p) and Spec[R].vanishing(j).contains(p))
            not Spec[R].vanishing(i).intersection(Spec[R].vanishing(j)).contains(p)
            not Spec[R].vanishing(i.sum(j)).contains(p)
            Spec[R].vanishing(i.sum(j)).c.contains(p)
        }
        Spec[R].vanishing(i.sum(j)).c.contains(p) =
            Spec[R].vanishing(i).c.union(Spec[R].vanishing(j).c).contains(p)
    }
}

/// `D(f) ⊆ D(1)`: every distinguished open is a subset of the whole spectrum.
theorem spec_basic_open_subset_one[R: CommRing](f: R) {
    Spec[R].basic_open(f).subset(Spec[R].basic_open(R.1))
} by {
    spec_basic_open_one[R]
}

/// `D(f * g) ⊆ D(f)`: if `f * g` is nonzero at a point then so is `f`.
theorem spec_basic_open_mul_subset_left[R: CommRing](f: R, g: R) {
    Spec[R].basic_open(f * g).subset(Spec[R].basic_open(f))
} by {
    spec_basic_open_mul(f, g)
    forall(p: Spec[R]) {
        intersection_contains_eq(Spec[R].basic_open(f), Spec[R].basic_open(g), p)
        if Spec[R].basic_open(f * g).contains(p) {
            Spec[R].basic_open(f).intersection(Spec[R].basic_open(g)).contains(p)
            Spec[R].basic_open(f).contains(p)
        }
    }
}

/// `D(f * g) ⊆ D(g)`: if `f * g` is nonzero at a point then so is `g`.
theorem spec_basic_open_mul_subset_right[R: CommRing](f: R, g: R) {
    Spec[R].basic_open(f * g).subset(Spec[R].basic_open(g))
} by {
    spec_basic_open_mul(f, g)
    forall(p: Spec[R]) {
        intersection_contains_eq(Spec[R].basic_open(f), Spec[R].basic_open(g), p)
        if Spec[R].basic_open(f * g).contains(p) {
            Spec[R].basic_open(f).intersection(Spec[R].basic_open(g)).contains(p)
            Spec[R].basic_open(g).contains(p)
        }
    }
}

/// True if a subset of the prime spectrum is Zariski-closed: it equals the
/// vanishing set of some ideal.
define spec_zariski_closed[R: CommRing](s: Set[Spec[R]]) -> Bool {
    exists(i: Ideal[R]) {
        s = Spec[R].vanishing(i)
    }
}

/// The empty set is Zariski-closed, witnessed by the unit ideal.
theorem spec_zariski_closed_empty[R: CommRing] {
    spec_zariski_closed(Set[Spec[R]].empty_set)
} by {
    spec_vanishing_unit[R]
    Set[Spec[R]].empty_set = Spec[R].vanishing(Ideal[R].unit)
}

/// The whole spectrum is Zariski-closed, witnessed by the zero ideal.
theorem spec_zariski_closed_universal[R: CommRing] {
    spec_zariski_closed(Set[Spec[R]].universal_set)
} by {
    spec_vanishing_zero[R]
    Set[Spec[R]].universal_set = Spec[R].vanishing(Ideal[R].zero)
}

/// Every vanishing set is Zariski-closed.
theorem spec_zariski_closed_vanishing[R: CommRing](i: Ideal[R]) {
    spec_zariski_closed(Spec[R].vanishing(i))
}

/// Zariski-closed sets are closed under binary intersection: `V(I) ∩ V(J) = V(I + J)`.
theorem spec_zariski_closed_inter[R: CommRing](s: Set[Spec[R]], t: Set[Spec[R]]) {
    spec_zariski_closed(s) and spec_zariski_closed(t)
        implies spec_zariski_closed(s.intersection(t))
} by {
    if spec_zariski_closed(s) and spec_zariski_closed(t) {
        let i: Ideal[R] satisfy { s = Spec[R].vanishing(i) }
        let j: Ideal[R] satisfy { t = Spec[R].vanishing(j) }
        spec_vanishing_sum(i, j)
        s.intersection(t) = Spec[R].vanishing(i.sum(j))
        exists(k: Ideal[R]) { s.intersection(t) = Spec[R].vanishing(k) }
    }
}

/// Zariski-closed sets are closed under binary union: `V(I) ∪ V(J) = V(I ∩ J)`.
theorem spec_zariski_closed_union[R: CommRing](s: Set[Spec[R]], t: Set[Spec[R]]) {
    spec_zariski_closed(s) and spec_zariski_closed(t)
        implies spec_zariski_closed(s.union(t))
} by {
    if spec_zariski_closed(s) and spec_zariski_closed(t) {
        let i: Ideal[R] satisfy { s = Spec[R].vanishing(i) }
        let j: Ideal[R] satisfy { t = Spec[R].vanishing(j) }
        spec_vanishing_union_eq_inter(i, j)
        s.union(t) = Spec[R].vanishing(i.intersection(j))
        exists(k: Ideal[R]) { s.union(t) = Spec[R].vanishing(k) }
    }
}

/// True if a subset of the prime spectrum is a distinguished basic open.
define spec_basic_open_family[R: CommRing](s: Set[Spec[R]]) -> Bool {
    exists(f: R) {
        s = Spec[R].basic_open(f)
    }
}

/// A distinguished basic open belongs to the basic-open family.
theorem spec_basic_open_family_member[R: CommRing](f: R) {
    spec_basic_open_family(Spec[R].basic_open(f))
} by {
    exists(g: R) {
        Spec[R].basic_open(f) = Spec[R].basic_open(g)
    }
}

/// The intersection of two distinguished basic opens is again a distinguished basic open.
theorem spec_basic_open_family_inter_member[R: CommRing](f: R, g: R) {
    spec_basic_open_family(Spec[R].basic_open(f).intersection(Spec[R].basic_open(g)))
} by {
    spec_basic_open_mul(f, g)
    Spec[R].basic_open(f * g) = Spec[R].basic_open(f).intersection(Spec[R].basic_open(g))
    Spec[R].basic_open(f).intersection(Spec[R].basic_open(g)) = Spec[R].basic_open(f * g)
    exists(h: R) {
        Spec[R].basic_open(f).intersection(Spec[R].basic_open(g)) = Spec[R].basic_open(h)
    }
}

/// The basic-open family is closed under binary intersections.
theorem spec_basic_open_family_inter[R: CommRing](s: Set[Spec[R]], t: Set[Spec[R]]) {
    spec_basic_open_family(s) and spec_basic_open_family(t)
        implies spec_basic_open_family(s.intersection(t))
} by {
    if spec_basic_open_family(s) and spec_basic_open_family(t) {
        let f: R satisfy { s = Spec[R].basic_open(f) }
        let g: R satisfy { t = Spec[R].basic_open(g) }
        spec_basic_open_family_inter_member(f, g)
        s.intersection(t) = Spec[R].basic_open(f).intersection(Spec[R].basic_open(g))
        spec_basic_open_family(s.intersection(t))
    }
}


/// True if a subset of the prime spectrum is Zariski-open: it is the
/// complement of a vanishing set.
define spec_zariski_open[R: CommRing](s: Set[Spec[R]]) -> Bool {
    exists(i: Ideal[R]) {
        s = Spec[R].vanishing(i).c
    }
}

/// Complements of vanishing sets are Zariski-open.
theorem spec_zariski_open_vanishing_compl[R: CommRing](i: Ideal[R]) {
    spec_zariski_open(Spec[R].vanishing(i).c)
} by {
    exists(j: Ideal[R]) {
        Spec[R].vanishing(i).c = Spec[R].vanishing(j).c
    }
}

/// The empty set is Zariski-open, as the complement of `V(0)`.
theorem spec_zariski_open_empty[R: CommRing] {
    spec_zariski_open(Set[Spec[R]].empty_set)
} by {
    spec_vanishing_zero_compl[R]
    Set[Spec[R]].empty_set = Spec[R].vanishing(Ideal[R].zero).c
    exists(i: Ideal[R]) {
        Set[Spec[R]].empty_set = Spec[R].vanishing(i).c
    }
}

/// The whole spectrum is Zariski-open, as the complement of `V(1)`.
theorem spec_zariski_open_universal[R: CommRing] {
    spec_zariski_open(Set[Spec[R]].universal_set)
} by {
    spec_vanishing_unit_compl[R]
    Set[Spec[R]].universal_set = Spec[R].vanishing(Ideal[R].unit).c
    exists(i: Ideal[R]) {
        Set[Spec[R]].universal_set = Spec[R].vanishing(i).c
    }
}

/// Every distinguished basic open is Zariski-open.
theorem spec_zariski_open_basic[R: CommRing](f: R) {
    spec_zariski_open(Spec[R].basic_open(f))
} by {
    spec_vanishing_principal_compl(f)
    Spec[R].basic_open(f) = Spec[R].vanishing(Ideal[R].principal(f)).c
    exists(i: Ideal[R]) {
        Spec[R].basic_open(f) = Spec[R].vanishing(i).c
    }
}

/// Zariski-open sets are closed under binary intersection.
theorem spec_zariski_open_inter[R: CommRing](s: Set[Spec[R]], t: Set[Spec[R]]) {
    spec_zariski_open(s) and spec_zariski_open(t)
        implies spec_zariski_open(s.intersection(t))
} by {
    if spec_zariski_open(s) and spec_zariski_open(t) {
        let i: Ideal[R] satisfy { s = Spec[R].vanishing(i).c }
        let j: Ideal[R] satisfy { t = Spec[R].vanishing(j).c }
        spec_vanishing_compl_inter(i, j)
        s.intersection(t) = Spec[R].vanishing(i.intersection(j)).c
        exists(k: Ideal[R]) {
            s.intersection(t) = Spec[R].vanishing(k).c
        }
    }
}

/// Zariski-open sets are closed under binary union.
theorem spec_zariski_open_union[R: CommRing](s: Set[Spec[R]], t: Set[Spec[R]]) {
    spec_zariski_open(s) and spec_zariski_open(t)
        implies spec_zariski_open(s.union(t))
} by {
    if spec_zariski_open(s) and spec_zariski_open(t) {
        let i: Ideal[R] satisfy { s = Spec[R].vanishing(i).c }
        let j: Ideal[R] satisfy { t = Spec[R].vanishing(j).c }
        spec_vanishing_sum_compl(i, j)
        s.union(t) = Spec[R].vanishing(i.sum(j)).c
        exists(k: Ideal[R]) {
            s.union(t) = Spec[R].vanishing(k).c
        }
    }
}

/// `Spec.of_maximal` is injective: distinct maximal ideals give distinct points.
theorem spec_of_maximal_inj[R: CommRing](m: MaximalIdeal[R], n: MaximalIdeal[R]) {
    Spec[R].of_maximal(m) = Spec[R].of_maximal(n) implies m.as_prime_ideal = n.as_prime_ideal
} by {
    if Spec[R].of_maximal(m) = Spec[R].of_maximal(n) {
        spec_of_maximal_point(m)
        spec_of_maximal_point(n)
        Spec[R].of_maximal(m).point = Spec[R].of_maximal(n).point
        m.as_prime_ideal = n.as_prime_ideal
    }
}
