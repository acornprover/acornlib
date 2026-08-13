/// Deep properties of quotient rings by prime and maximal ideals: the quotient
/// of a commutative ring by a prime ideal has no zero divisors, and the
/// quotient by a maximal ideal has every nonzero element a unit.

from comm_ring import CommRing
from algebra.add_group import inverse_add
from algebra.ring.ideal import Ideal, is_prime_ideal, is_prime_ideal_mem, is_maximal_ideal,
    is_ideal_add, is_ideal_neg, ideal_contains_zero,
    principal_ideal, principal_ideal_witness, ideal_sum, maximal_ideal_sum_principal_contains_one
from algebra.ring.ideal_quotient import ideal_quotient_project, ideal_quotient_zero, ideal_quotient_project_eq_zero_iff_contains,
    ideal_quotient_project_eq_iff_contains_sub, ideal_quotient_project_eq_of_contains_sub
from algebra.ring.quotient_ring import IdealQuotient, ideal_quotient_mk_type, ideal_quotient_type_mul,
    ideal_quotient_type_zero, ideal_quotient_type_one,
    ideal_quotient_mk_type_val, ideal_quotient_type_mul_mk, ideal_quotient_type_zero_mk,
    ideal_quotient_type_one_mk, ideal_quotient_type_ext, ideal_quotient_type_zero_val, ideal_quotient_type_one_val

from nat import Nat
numerals Nat

/// The projection of the ring zero is the quotient zero, at the level of values.
theorem ideal_quotient_zero_projection_local[R: CommRing](i: Ideal[R]) {
    ideal_quotient_zero(i) = ideal_quotient_project(i, R.0)
} by {
}

/// Ideals are closed under negation.
theorem ideal_neg_mem_local[R: CommRing](i: Ideal[R], a: R) {
    i.contains(a) implies i.contains(-a)
} by {
    if i.contains(a) {
        is_ideal_neg(i.contains, a)
    }
}

/// In a maximal ideal quotient, membership of `(r * a) - 1` in the ideal
/// follows from `1 = x + r * a` with `x` in the ideal.
theorem maximal_ideal_quotient_unit_contains_sub[R: CommRing](i: Ideal[R], x: R, r: R, a: R) {
    i.contains(x) and R.1 = x + r * a implies i.contains((r * a) - R.1)
} by {
    if i.contains(x) and R.1 = x + r * a {
        i.contains(x)
        R.1 = x + r * a
        ideal_contains_zero(i)
        i.contains(R.0)
        (x + r * a) - R.1 = R.0
        i.contains((x + r * a) - R.1)
        ideal_neg_mem_local(i, x)
        i.contains(-x)
        (r * a) - (x + r * a) = (r * a) + (-(x + r * a))
        inverse_add(x, r * a)
        -(x + r * a) = -(r * a) + -x
        (r * a) + (-(r * a) + -x) = ((r * a) + -(r * a)) + (-x)
        (r * a) + -(r * a) = R.0
        ((r * a) + -(r * a)) + (-x) = R.0 + (-x)
        R.0 + (-x) = -x
        (r * a) - (x + r * a) = -x
        i.contains((r * a) - (x + r * a))
        is_ideal_add(i.contains, (r * a) - (x + r * a), (x + r * a) - R.1)
        i.contains(((r * a) - (x + r * a)) + ((x + r * a) - R.1))
        ((r * a) - (x + r * a)) + ((x + r * a) - R.1) = (r * a) - R.1
        i.contains((r * a) - R.1)
    }
}

/// In the quotient by a prime ideal, a product of two representatives is zero
/// only if one of them is zero: the quotient has no zero divisors.
theorem prime_ideal_quotient_no_zero_divisors[R: CommRing](i: Ideal[R], a: R, b: R) {
    is_prime_ideal(i.contains) and
    ideal_quotient_type_mul(i, ideal_quotient_mk_type(i, a), ideal_quotient_mk_type(i, b)) =
        ideal_quotient_type_zero(i)
    implies ideal_quotient_mk_type(i, a) = ideal_quotient_type_zero(i) or
            ideal_quotient_mk_type(i, b) = ideal_quotient_type_zero(i)
} by {
    if is_prime_ideal(i.contains) and
        ideal_quotient_type_mul(i, ideal_quotient_mk_type(i, a), ideal_quotient_mk_type(i, b)) =
            ideal_quotient_type_zero(i) {
        ideal_quotient_type_mul_mk(i, a, b)
        ideal_quotient_type_mul(i, ideal_quotient_mk_type(i, a), ideal_quotient_mk_type(i, b)) =
            ideal_quotient_mk_type(i, a * b)
        ideal_quotient_mk_type(i, a * b) = ideal_quotient_type_zero(i)
        ideal_quotient_type_zero_mk(i)
        ideal_quotient_type_zero(i) = ideal_quotient_mk_type(i, R.0)
        ideal_quotient_mk_type(i, a * b) = ideal_quotient_mk_type(i, R.0)
        ideal_quotient_mk_type_val(i, a * b)
        ideal_quotient_mk_type_val(i, R.0)
        ideal_quotient_project(i, a * b) = ideal_quotient_project(i, R.0)
        ideal_quotient_project_eq_iff_contains_sub(i, a * b, R.0)
        i.contains(a * b - R.0)
        i.contains(a * b)
        is_prime_ideal_mem(i.contains, a, b)
        i.contains(a) or i.contains(b)
        if i.contains(a) {
            ideal_quotient_project_eq_zero_iff_contains(i, a)
            ideal_quotient_project(i, a) = ideal_quotient_zero(i)
            ideal_quotient_mk_type_val(i, a)
            ideal_quotient_mk_type(i, a).val = ideal_quotient_project(i, a)
            ideal_quotient_type_zero_val(i)
            ideal_quotient_type_zero(i).val = ideal_quotient_zero(i)
            ideal_quotient_mk_type(i, a).val = ideal_quotient_type_zero(i).val
            ideal_quotient_type_ext(i, ideal_quotient_mk_type(i, a), ideal_quotient_type_zero(i))
            ideal_quotient_mk_type(i, a) = ideal_quotient_type_zero(i)
        }
        if i.contains(b) {
            ideal_quotient_project_eq_zero_iff_contains(i, b)
            ideal_quotient_project(i, b) = ideal_quotient_zero(i)
            ideal_quotient_mk_type_val(i, b)
            ideal_quotient_mk_type(i, b).val = ideal_quotient_project(i, b)
            ideal_quotient_type_zero_val(i)
            ideal_quotient_type_zero(i).val = ideal_quotient_zero(i)
            ideal_quotient_mk_type(i, b).val = ideal_quotient_type_zero(i).val
            ideal_quotient_type_ext(i, ideal_quotient_mk_type(i, b), ideal_quotient_type_zero(i))
            ideal_quotient_mk_type(i, b) = ideal_quotient_type_zero(i)
        }
        ideal_quotient_mk_type(i, a) = ideal_quotient_type_zero(i) or
            ideal_quotient_mk_type(i, b) = ideal_quotient_type_zero(i)
    }
}

/// In the quotient by a maximal ideal, every nonzero element is a unit: for
/// every representative `a` outside the ideal there is a representative `b`
/// with `[a] * [b] = [1]`.
theorem maximal_ideal_quotient_nonzero_is_unit[R: CommRing](i: Ideal[R], a: R) {
    is_maximal_ideal(i.contains) and
    ideal_quotient_mk_type(i, a) != ideal_quotient_type_zero(i)
    implies exists(b: R) {
        ideal_quotient_type_mul(i, ideal_quotient_mk_type(i, a), ideal_quotient_mk_type(i, b)) =
            ideal_quotient_type_one(i)
    }
} by {
    if is_maximal_ideal(i.contains) and
        ideal_quotient_mk_type(i, a) != ideal_quotient_type_zero(i) {
        ideal_quotient_mk_type_val(i, a)
        ideal_quotient_mk_type(i, a).val = ideal_quotient_project(i, a)
        ideal_quotient_type_zero_mk(i)
        ideal_quotient_type_zero(i) = ideal_quotient_mk_type(i, R.0)
        ideal_quotient_mk_type_val(i, R.0)
        ideal_quotient_mk_type(i, R.0).val = ideal_quotient_project(i, R.0)
        if ideal_quotient_project(i, a) = ideal_quotient_project(i, R.0) {
            ideal_quotient_mk_type(i, a).val = ideal_quotient_mk_type(i, R.0).val
            ideal_quotient_type_ext(i, ideal_quotient_mk_type(i, a), ideal_quotient_mk_type(i, R.0))
            ideal_quotient_mk_type(i, a) = ideal_quotient_mk_type(i, R.0)
            ideal_quotient_mk_type(i, a) = ideal_quotient_type_zero(i)
            false
        }
        ideal_quotient_project(i, a) != ideal_quotient_project(i, R.0)
        ideal_quotient_project_eq_zero_iff_contains(i, a)
        ideal_quotient_zero_projection_local(i)
        ideal_quotient_project(i, a) != ideal_quotient_zero(i)
        not i.contains(a)
        maximal_ideal_sum_principal_contains_one(i.contains, a)
        ideal_sum(i.contains, principal_ideal[R](a), R.1)
        let (x: R, y: R) satisfy {
            i.contains(x) and principal_ideal[R](a, y) and R.1 = x + y
        }
        principal_ideal_witness(a, y)
        let r: R satisfy { y = r * a }
        R.1 = x + r * a
        maximal_ideal_quotient_unit_contains_sub(i, x, r, a)
        i.contains((r * a) - R.1)
        ideal_quotient_project_eq_of_contains_sub(i, r * a, R.1)
        ideal_quotient_project(i, r * a) = ideal_quotient_project(i, R.1)
        ideal_quotient_mk_type_val(i, r * a)
        ideal_quotient_mk_type(i, r * a).val = ideal_quotient_project(i, r * a)
        ideal_quotient_mk_type_val(i, R.1)
        ideal_quotient_mk_type(i, R.1).val = ideal_quotient_project(i, R.1)
        ideal_quotient_mk_type(i, r * a).val = ideal_quotient_mk_type(i, R.1).val
        ideal_quotient_type_ext(i, ideal_quotient_mk_type(i, r * a), ideal_quotient_mk_type(i, R.1))
        ideal_quotient_mk_type(i, r * a) = ideal_quotient_mk_type(i, R.1)
        ideal_quotient_type_one_mk(i)
        ideal_quotient_type_one(i) = ideal_quotient_mk_type(i, R.1)
        ideal_quotient_mk_type(i, r * a) = ideal_quotient_type_one(i)
        ideal_quotient_type_mul_mk(i, a, r)
        ideal_quotient_type_mul(i, ideal_quotient_mk_type(i, a), ideal_quotient_mk_type(i, r)) =
            ideal_quotient_mk_type(i, a * r)
        a * r = r * a
        ideal_quotient_mk_type(i, a * r) = ideal_quotient_mk_type(i, r * a)
        ideal_quotient_type_mul(i, ideal_quotient_mk_type(i, a), ideal_quotient_mk_type(i, r)) =
            ideal_quotient_type_one(i)
        exists(b: R) {
            ideal_quotient_type_mul(i, ideal_quotient_mk_type(i, a), ideal_quotient_mk_type(i, b)) =
                ideal_quotient_type_one(i)
        }
    }
}
