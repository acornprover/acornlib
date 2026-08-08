from comm_ring import CommRing
from data.basic.set import Set
from algebra.ring.ideal import Ideal, bundled_ideal_subset, bundled_ideal_subset_trans,
    set_subset_ideal, ideal_closure_contains_of_set_contains,
    ideal_closure_le_iff_set_subset, ideal_contains_mul_left,
    ideal_contains_mul_right, bundled_ideal_inter_contains_eq,
    bundled_ideal_inter_subset_left, bundled_ideal_inter_subset_right

/// The pairwise-product generators for the product of two ideals.
define ideal_product_generators_contains[R: CommRing](i: Ideal[R], j: Ideal[R], x: R) -> Bool {
    exists(a: R, b: R) {
        i.contains(a) and j.contains(b) and x = a * b
    }
}

/// The set of pairwise-product generators for the product of two ideals.
define ideal_product_generators[R: CommRing](i: Ideal[R], j: Ideal[R]) -> Set[R] {
    Set[R].new(ideal_product_generators_contains(i, j))
}

/// Membership in the generator set means being a product of one element from each ideal.
theorem ideal_product_generators_contains_eq[R: CommRing](i: Ideal[R], j: Ideal[R], x: R) {
    ideal_product_generators(i, j).contains(x) = exists(a: R, b: R) {
        i.contains(a) and j.contains(b) and x = a * b
    }
} by {
}

/// A product of one element from each ideal is a generator.
theorem ideal_product_generators_contains_mul[R: CommRing](i: Ideal[R], j: Ideal[R], a: R, b: R) {
    i.contains(a) and j.contains(b) implies ideal_product_generators(i, j).contains(a * b)
} by {
    if i.contains(a) and j.contains(b) {
        a * b = a * b
        ideal_product_generators_contains_eq(i, j, a * b)
        ideal_product_generators(i, j).contains(a * b)
    }
}

/// The product of two ideals: the ideal generated by pairwise products.
define ideal_product[R: CommRing](i: Ideal[R], j: Ideal[R]) -> Ideal[R] {
    Ideal[R].closure(ideal_product_generators(i, j))
}

/// A product of one element from each ideal belongs to the product ideal.
theorem ideal_product_contains_mul[R: CommRing](i: Ideal[R], j: Ideal[R], a: R, b: R) {
    i.contains(a) and j.contains(b) implies ideal_product(i, j).contains(a * b)
} by {
    if i.contains(a) and j.contains(b) {
        ideal_product_generators_contains_mul(i, j, a, b)
        ideal_closure_contains_of_set_contains(ideal_product_generators(i, j), a * b)
        ideal_product(i, j).contains(a * b)
    }
}

/// Every generator belongs to the product ideal it generates.
theorem ideal_product_contains_of_generator[R: CommRing](i: Ideal[R], j: Ideal[R], x: R) {
    ideal_product_generators(i, j).contains(x) implies ideal_product(i, j).contains(x)
} by {
    if ideal_product_generators(i, j).contains(x) {
        ideal_closure_contains_of_set_contains(ideal_product_generators(i, j), x)
        ideal_product(i, j).contains(x)
    }
}

/// A generator witness gives membership in the product ideal.
theorem ideal_product_contains_of_generator_witness[R: CommRing](
    i: Ideal[R], j: Ideal[R], a: R, b: R, x: R
) {
    i.contains(a) and j.contains(b) and x = a * b implies ideal_product(i, j).contains(x)
} by {
    if i.contains(a) and j.contains(b) and x = a * b {
        ideal_product_contains_mul(i, j, a, b)
        ideal_product(i, j).contains(x)
    }
}

/// The product ideal is contained in an ideal when all pairwise products are contained in it.
theorem ideal_product_subset_of_mul_mem[R: CommRing](i: Ideal[R], j: Ideal[R], c: Ideal[R]) {
    (forall(a: R, b: R) { i.contains(a) and j.contains(b) implies c.contains(a * b) })
        implies bundled_ideal_subset(ideal_product(i, j), c)
} by {
    if forall(a: R, b: R) { i.contains(a) and j.contains(b) implies c.contains(a * b) } {
        forall(x: R) {
            if ideal_product_generators(i, j).contains(x) {
                ideal_product_generators_contains_eq(i, j, x)
                let (a: R, b: R) satisfy {
                    i.contains(a) and j.contains(b) and x = a * b
                }
                c.contains(a * b)
                c.contains(x)
            }
        }
        set_subset_ideal(ideal_product_generators(i, j), c)
        ideal_closure_le_iff_set_subset(ideal_product_generators(i, j), c)
        bundled_ideal_subset(ideal_product(i, j), c)
    }
}

/// The product ideal satisfies the expected universal property.
theorem ideal_product_subset_iff_mul_mem[R: CommRing](i: Ideal[R], j: Ideal[R], c: Ideal[R]) {
    bundled_ideal_subset(ideal_product(i, j), c) =
        forall(a: R, b: R) { i.contains(a) and j.contains(b) implies c.contains(a * b) }
} by {
    if bundled_ideal_subset(ideal_product(i, j), c) {
        forall(a: R, b: R) {
            if i.contains(a) and j.contains(b) {
                ideal_product_contains_mul(i, j, a, b)
                bundled_ideal_subset(ideal_product(i, j), c) = forall(x: R) {
                    ideal_product(i, j).contains(x) implies c.contains(x)
                }
                c.contains(a * b)
            }
        }
    }
    if forall(a: R, b: R) { i.contains(a) and j.contains(b) implies c.contains(a * b) } {
        ideal_product_subset_of_mul_mem(i, j, c)
        bundled_ideal_subset(ideal_product(i, j), c)
    }
}

/// Ideal product is monotone in both arguments.
theorem ideal_product_monotone[R: CommRing](i: Ideal[R], j: Ideal[R], i_target: Ideal[R], j_target: Ideal[R]) {
    bundled_ideal_subset(i, i_target) and bundled_ideal_subset(j, j_target) implies
        bundled_ideal_subset(ideal_product(i, j), ideal_product(i_target, j_target))
} by {
    if bundled_ideal_subset(i, i_target) and bundled_ideal_subset(j, j_target) {
        forall(a: R, b: R) {
            if i.contains(a) and j.contains(b) {
                bundled_ideal_subset(i, i_target) = forall(x: R) {
                    i.contains(x) implies i_target.contains(x)
                }
                bundled_ideal_subset(j, j_target) = forall(x: R) {
                    j.contains(x) implies j_target.contains(x)
                }
                i_target.contains(a)
                j_target.contains(b)
                ideal_product_contains_mul(i_target, j_target, a, b)
                ideal_product(i_target, j_target).contains(a * b)
            }
        }
        ideal_product_subset_of_mul_mem(i, j, ideal_product(i_target, j_target))
        bundled_ideal_subset(ideal_product(i, j), ideal_product(i_target, j_target))
    }
}

/// The product of two ideals is contained in their intersection.
theorem ideal_product_subset_intersection[R: CommRing](i: Ideal[R], j: Ideal[R]) {
    bundled_ideal_subset(ideal_product(i, j), i.intersection(j))
} by {
    forall(a: R, b: R) {
        if i.contains(a) and j.contains(b) {
            ideal_contains_mul_right(i, a, b)
            ideal_contains_mul_left(j, a, b)
            bundled_ideal_inter_contains_eq(i, j, a * b)
            i.intersection(j).contains(a * b)
        }
    }
    ideal_product_subset_of_mul_mem(i, j, i.intersection(j))
}

/// The product of two ideals is contained in the left factor.
theorem ideal_product_subset_left[R: CommRing](i: Ideal[R], j: Ideal[R]) {
    bundled_ideal_subset(ideal_product(i, j), i)
} by {
    ideal_product_subset_intersection(i, j)
    bundled_ideal_inter_subset_left(i, j)
    bundled_ideal_subset_trans(ideal_product(i, j), i.intersection(j), i)
}

/// The product of two ideals is contained in the right factor.
theorem ideal_product_subset_right[R: CommRing](i: Ideal[R], j: Ideal[R]) {
    bundled_ideal_subset(ideal_product(i, j), j)
} by {
    ideal_product_subset_intersection(i, j)
    bundled_ideal_inter_subset_right(i, j)
    bundled_ideal_subset_trans(ideal_product(i, j), i.intersection(j), j)
}

/// Membership in a product ideal implies membership in the left factor.
theorem ideal_product_contains_implies_left[R: CommRing](i: Ideal[R], j: Ideal[R], x: R) {
    ideal_product(i, j).contains(x) implies i.contains(x)
} by {
    if ideal_product(i, j).contains(x) {
        ideal_product_subset_left(i, j)
        bundled_ideal_subset(ideal_product(i, j), i) = forall(y: R) {
            ideal_product(i, j).contains(y) implies i.contains(y)
        }
        i.contains(x)
    }
}

/// Membership in a product ideal implies membership in the right factor.
theorem ideal_product_contains_implies_right[R: CommRing](i: Ideal[R], j: Ideal[R], x: R) {
    ideal_product(i, j).contains(x) implies j.contains(x)
} by {
    if ideal_product(i, j).contains(x) {
        ideal_product_subset_right(i, j)
        bundled_ideal_subset(ideal_product(i, j), j) = forall(y: R) {
            ideal_product(i, j).contains(y) implies j.contains(y)
        }
        j.contains(x)
    }
}
