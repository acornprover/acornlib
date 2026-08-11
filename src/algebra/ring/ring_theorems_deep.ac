/// Deep ring-theory theorems: kernel-ideal facts for ring homomorphisms,
/// ideal closure combinations, quotient-ring facts built on the bundled
/// `IdealQuotient` type, and commutative-ring arithmetic identities such as
/// the square of a difference and the difference of squares.
///
/// The heavy machinery (bundled ideals, ideal quotients, canonical
/// projection, first-isomorphism ingredients) lives in `ideal.ac`,
/// `quotient_ring.ac`, and `ideal_quotient.ac`; this file combines and
/// extends it with the theorems below.

from comm_ring import CommRing
from nat import Nat
from algebra.ring.ring import Ring, mul_neg_right, mul_neg_neg
from algebra.ring.ring_hom import RingHom, ring_hom_add, ring_hom_neg, ring_hom_mul, ring_hom_zero,
    ring_hom_to_monoid_hom, compose_ring_hom, compose_ring_hom_hom
from algebra.ring.ideal import Ideal, is_ideal, ring_hom_kernel, ring_hom_kernel_contains_eq,
    ring_hom_kernel_contains_of_maps_to_zero, ring_hom_maps_to_zero_of_kernel_contains,
    ideal_preimage, ideal_preimage_contains_eq, ideal_eq_of_contains_at_eq,
    bundled_ideal_inter_contains_eq, bundled_ideal_sum_contains, bundled_ideal_sum_contains_eq,
    ideal_contains_mul_right
from data.basic.functions import compose
from algebra.ring.ideal_quotient import ideal_quotient_project, ideal_quotient_project_eq_iff_contains_sub,
    ideal_quotient_project_eq_zero_iff_contains, ideal_quotient_project_eq_mk, ideal_quotient_zero_projection,
    ideal_quotient_relation, ideal_quotient_zero
from data.basic.equivalence import quotient_over_mk
from algebra.ring.quotient_ring import IdealQuotient, ideal_quotient_mk_type, ideal_quotient_mk_type_val,
    ideal_quotient_type_ext, ideal_quotient_typeclass_zero_eq, ideal_quotient_type_zero_mk,
    ideal_quotient_type_neg_mk, ideal_quotient_typeclass_neg_eq, ideal_quotient_type_neg,
    ideal_quotient_kernel_induced_project, ideal_quotient_kernel_induced_injective
from data.basic.quotient_algebra import ring_hom_kernel_quotient_induced
from algebra.hom_kernel_injective import ring_hom_eq_iff_sub_zero
from algebra.units import Unit, is_monoid_unit, monoid_hom_maps_monoid_unit, monoid_hom_unit,
    monoid_hom_unit_val, monoid_hom_unit_inv
from algebra.comm_ring_unit import is_unit_of_monoid_unit, is_monoid_unit_of_is_unit
from algebra.zero import Zero
from algebra.add_comm_monoid_rearrange import add_shift_left

// ---------------------------------------------------------------------------
// Section 1: Ring homomorphisms
// ---------------------------------------------------------------------------

/// A ring homomorphism preserves subtraction.
theorem ring_hom_sub[R: Ring, S: Ring](f: RingHom[R, S], a: R, b: R) {
    f.hom(a - b) = f.hom(a) - f.hom(b)
} by {
    a - b = a + -b
    ring_hom_add(f, a, -b)
    f.hom(a - b) = f.hom(a) + f.hom(-b)
    ring_hom_neg(f, b)
    f.hom(-b) = -f.hom(b)
    f.hom(a) - f.hom(b) = f.hom(a) + -f.hom(b)
}

/// A ring homomorphism carries the value of a unit to the value of the image unit.
theorem ring_hom_unit_val[R: Ring, S: Ring](f: RingHom[R, S], u: Unit[R]) {
    monoid_hom_unit(ring_hom_to_monoid_hom(f), u).val = f.hom(u.val)
} by {
    monoid_hom_unit_val(ring_hom_to_monoid_hom(f), u)
    monoid_hom_unit(ring_hom_to_monoid_hom(f), u).val = f.hom(u.val)
}

/// A ring homomorphism carries the inverse of a unit to the inverse of the image unit.
theorem ring_hom_unit_inv[R: Ring, S: Ring](f: RingHom[R, S], u: Unit[R]) {
    monoid_hom_unit(ring_hom_to_monoid_hom(f), u).inv = f.hom(u.inv)
} by {
    monoid_hom_unit_inv(ring_hom_to_monoid_hom(f), u)
    monoid_hom_unit(ring_hom_to_monoid_hom(f), u).inv = f.hom(u.inv)
}

/// A ring homomorphism between commutative rings maps units to units.
theorem ring_hom_maps_unit[R: CommRing, S: CommRing](f: RingHom[R, S], a: R) {
    a.is_unit implies f.hom(a).is_unit
} by {
    if a.is_unit {
        is_monoid_unit_of_is_unit(a)
        is_monoid_unit(a)
        monoid_hom_maps_monoid_unit(ring_hom_to_monoid_hom(f), a)
        is_monoid_unit(f.hom(a))
        is_unit_of_monoid_unit(f.hom(a))
        f.hom(a).is_unit
    }
}

// ---------------------------------------------------------------------------
// Section 2: Kernels of ring homomorphisms are ideals
// ---------------------------------------------------------------------------

/// Zero lies in the kernel of a ring homomorphism.
theorem ring_hom_kernel_contains_zero[R: CommRing, S: CommRing](f: RingHom[R, S]) {
    ring_hom_kernel(f).contains(R.0)
} by {
    ring_hom_zero(f)
    f.hom(R.0) = S.0
    ring_hom_kernel_contains_of_maps_to_zero(f, R.0)
    ring_hom_kernel(f).contains(R.0)
}

/// The kernel of a ring homomorphism is closed under addition.
theorem ring_hom_kernel_contains_add[R: CommRing, S: CommRing](f: RingHom[R, S], a: R, b: R) {
    ring_hom_kernel(f).contains(a) and ring_hom_kernel(f).contains(b) implies
        ring_hom_kernel(f).contains(a + b)
} by {
    if ring_hom_kernel(f).contains(a) and ring_hom_kernel(f).contains(b) {
        ring_hom_maps_to_zero_of_kernel_contains(f, a)
        ring_hom_maps_to_zero_of_kernel_contains(f, b)
        f.hom(a) = S.0
        f.hom(b) = S.0
        ring_hom_add(f, a, b)
        f.hom(a + b) = f.hom(a) + f.hom(b)
        S.0 + S.0 = S.0
        f.hom(a + b) = S.0
        ring_hom_kernel_contains_of_maps_to_zero(f, a + b)
        ring_hom_kernel(f).contains(a + b)
    }
}

/// The kernel of a ring homomorphism absorbs multiplication.
theorem ring_hom_kernel_contains_mul[R: CommRing, S: CommRing](f: RingHom[R, S], r: R, a: R) {
    ring_hom_kernel(f).contains(a) implies ring_hom_kernel(f).contains(r * a)
} by {
    if ring_hom_kernel(f).contains(a) {
        ring_hom_maps_to_zero_of_kernel_contains(f, a)
        f.hom(a) = S.0
        ring_hom_mul(f, r, a)
        f.hom(r * a) = f.hom(r) * f.hom(a)
        f.hom(r) * S.0 = S.0
        f.hom(r * a) = S.0
        ring_hom_kernel_contains_of_maps_to_zero(f, r * a)
        ring_hom_kernel(f).contains(r * a)
    }
}

/// The kernel of a ring homomorphism is an ideal.
theorem ring_hom_kernel_is_ideal[R: CommRing, S: CommRing](f: RingHom[R, S]) {
    is_ideal(ring_hom_kernel(f).contains)
} by {
    is_ideal(ring_hom_kernel(f).contains)
}

/// Kernel membership of a difference is equality of images.
theorem ring_hom_kernel_contains_sub_iff[R: CommRing, S: CommRing](f: RingHom[R, S], a: R, b: R) {
    ring_hom_kernel(f).contains(a - b) = (f.hom(a) = f.hom(b))
} by {
    ring_hom_kernel_contains_eq(f, a - b)
    ring_hom_kernel(f).contains(a - b) = (f.hom(a - b) = S.0)
    ring_hom_eq_iff_sub_zero(f, b, a)
    (f.hom(b) = f.hom(a)) = (f.hom(a - b) = S.0)
    ring_hom_kernel(f).contains(a - b) = (f.hom(b) = f.hom(a))
    f.hom(b) = f.hom(a) = (f.hom(a) = f.hom(b))
}

/// The kernel of a composition is the preimage of the kernel.
theorem ring_hom_kernel_of_compose[R: CommRing, S: CommRing, T: CommRing](
    f: RingHom[S, T],
    g: RingHom[R, S]
) {
    ring_hom_kernel(compose_ring_hom(f, g)) = ideal_preimage(g, ring_hom_kernel(f))
} by {
    forall(x: R) {
        ring_hom_kernel_contains_eq(compose_ring_hom(f, g), x)
        ring_hom_kernel(compose_ring_hom(f, g)).contains(x) = (compose_ring_hom(f, g).hom(x) = T.0)
        compose_ring_hom_hom(f, g)
        compose_ring_hom(f, g).hom = compose(f.hom, g.hom)
        compose(f.hom, g.hom)(x) = f.hom(g.hom(x))
        compose_ring_hom(f, g).hom(x) = f.hom(g.hom(x))
        ring_hom_kernel(compose_ring_hom(f, g)).contains(x) = (f.hom(g.hom(x)) = T.0)
        ideal_preimage_contains_eq(g, ring_hom_kernel(f), x)
        ideal_preimage(g, ring_hom_kernel(f)).contains(x) = ring_hom_kernel(f).contains(g.hom(x))
        ring_hom_kernel_contains_eq(f, g.hom(x))
        ring_hom_kernel(f).contains(g.hom(x)) = (f.hom(g.hom(x)) = T.0)
        ideal_preimage(g, ring_hom_kernel(f)).contains(x) = (f.hom(g.hom(x)) = T.0)
        ring_hom_kernel(compose_ring_hom(f, g)).contains(x) =
            ideal_preimage(g, ring_hom_kernel(f)).contains(x)
    }
    ideal_eq_of_contains_at_eq(ring_hom_kernel(compose_ring_hom(f, g)),
        ideal_preimage(g, ring_hom_kernel(f)))
}

// ---------------------------------------------------------------------------
// Section 3: Ideal closure combinations
// ---------------------------------------------------------------------------

/// The product of two ideal elements lies in the ideal.
theorem ideal_contains_mul_of_contains[R: CommRing](i: Ideal[R], a: R, b: R) {
    i.contains(a) and i.contains(b) implies i.contains(a * b)
} by {
    if i.contains(a) and i.contains(b) {
        ideal_contains_mul_right(i, a, b)
        i.contains(a * b)
    }
}

/// An element of both ideals lies in their intersection.
theorem ideal_inter_contains_of_contains[R: CommRing](i: Ideal[R], j: Ideal[R], a: R) {
    i.contains(a) and j.contains(a) implies i.intersection(j).contains(a)
} by {
    if i.contains(a) and j.contains(a) {
        bundled_ideal_inter_contains_eq(i, j, a)
        i.intersection(j).contains(a)
    }
}

/// A sum of an element of `i` and an element of `j` lies in the sum ideal.
theorem ideal_sum_contains_add[R: CommRing](i: Ideal[R], j: Ideal[R], a: R, b: R) {
    i.contains(a) and j.contains(b) implies i.sum(j).contains(a + b)
} by {
    if i.contains(a) and j.contains(b) {
        bundled_ideal_sum_contains_eq(i, j, a + b)
        a + b = a + b
        bundled_ideal_sum_contains(i, j, a + b)
        i.sum(j).contains(a + b)
    }
}

// ---------------------------------------------------------------------------
// Section 4: Quotient rings
// ---------------------------------------------------------------------------

/// Equality of cosets in `R/I` is membership of the difference in the ideal.
theorem ideal_quotient_mk_type_eq_iff_contains_sub[R: CommRing](i: Ideal[R], a: R, b: R) {
    (ideal_quotient_mk_type(i, a) = ideal_quotient_mk_type(i, b)) = i.contains(a - b)
} by {
    if ideal_quotient_mk_type(i, a) = ideal_quotient_mk_type(i, b) {
        ideal_quotient_mk_type(i, a).val = ideal_quotient_mk_type(i, b).val
        ideal_quotient_mk_type_val(i, a)
        ideal_quotient_mk_type_val(i, b)
        ideal_quotient_project(i, a) = ideal_quotient_project(i, b)
        ideal_quotient_project_eq_iff_contains_sub(i, a, b)
        i.contains(a - b)
    }
    if i.contains(a - b) {
        ideal_quotient_project_eq_iff_contains_sub(i, a, b)
        ideal_quotient_project(i, a) = ideal_quotient_project(i, b)
        ideal_quotient_mk_type_val(i, a)
        ideal_quotient_mk_type_val(i, b)
        ideal_quotient_mk_type(i, a).val = ideal_quotient_mk_type(i, b).val
        ideal_quotient_type_ext(i, ideal_quotient_mk_type(i, a), ideal_quotient_mk_type(i, b))
        ideal_quotient_mk_type(i, a) = ideal_quotient_mk_type(i, b)
    }
    (ideal_quotient_mk_type(i, a) = ideal_quotient_mk_type(i, b)) = i.contains(a - b)
}

/// A coset in `R/I` is zero exactly when the representative lies in the ideal.
theorem ideal_quotient_mk_type_eq_zero_iff_contains[R: CommRing](i: Ideal[R], a: R) {
    (ideal_quotient_mk_type(i, a) = Zero.0[IdealQuotient[R, i]]) = i.contains(a)
} by {
    if ideal_quotient_mk_type(i, a) = Zero.0[IdealQuotient[R, i]] {
        ideal_quotient_typeclass_zero_eq(i)
        ideal_quotient_type_zero_mk(i)
        Zero.0[IdealQuotient[R, i]] = ideal_quotient_mk_type(i, R.0)
        ideal_quotient_mk_type(i, a) = ideal_quotient_mk_type(i, R.0)
        ideal_quotient_mk_type_val(i, a)
        ideal_quotient_mk_type_val(i, R.0)
        ideal_quotient_project(i, a) = ideal_quotient_project(i, R.0)
        ideal_quotient_project_eq_mk(i, R.0)
        ideal_quotient_project(i, R.0) = quotient_over_mk(ideal_quotient_relation(i), R.0)
        ideal_quotient_zero_projection(i)
        ideal_quotient_zero(i) = quotient_over_mk(ideal_quotient_relation(i), R.0)
        ideal_quotient_project(i, R.0) = ideal_quotient_zero(i)
        ideal_quotient_project(i, a) = ideal_quotient_zero(i)
        ideal_quotient_project_eq_zero_iff_contains(i, a)
        i.contains(a)
    }
    if i.contains(a) {
        ideal_quotient_project_eq_zero_iff_contains(i, a)
        ideal_quotient_project(i, a) = ideal_quotient_zero(i)
        ideal_quotient_project_eq_mk(i, R.0)
        ideal_quotient_project(i, R.0) = quotient_over_mk(ideal_quotient_relation(i), R.0)
        ideal_quotient_zero_projection(i)
        ideal_quotient_zero(i) = quotient_over_mk(ideal_quotient_relation(i), R.0)
        ideal_quotient_project(i, R.0) = ideal_quotient_zero(i)
        ideal_quotient_mk_type_val(i, a)
        ideal_quotient_mk_type(i, a).val = ideal_quotient_project(i, a)
        ideal_quotient_mk_type_val(i, R.0)
        ideal_quotient_mk_type(i, R.0).val = ideal_quotient_project(i, R.0)
        ideal_quotient_mk_type(i, a).val = ideal_quotient_mk_type(i, R.0).val
        ideal_quotient_type_ext(i, ideal_quotient_mk_type(i, a), ideal_quotient_mk_type(i, R.0))
        ideal_quotient_mk_type(i, a) = ideal_quotient_mk_type(i, R.0)
        ideal_quotient_type_zero_mk(i)
        ideal_quotient_typeclass_zero_eq(i)
        ideal_quotient_mk_type(i, R.0) = Zero.0[IdealQuotient[R, i]]
        ideal_quotient_mk_type(i, a) = Zero.0[IdealQuotient[R, i]]
    }
    (ideal_quotient_mk_type(i, a) = Zero.0[IdealQuotient[R, i]]) = i.contains(a)
}

/// An ideal element projects to the zero coset.
theorem ideal_quotient_mk_type_eq_zero_of_contains[R: CommRing](i: Ideal[R], a: R) {
    i.contains(a) implies ideal_quotient_mk_type(i, a) = Zero.0[IdealQuotient[R, i]]
} by {
    if i.contains(a) {
        ideal_quotient_mk_type_eq_zero_iff_contains(i, a)
        ideal_quotient_mk_type(i, a) = Zero.0[IdealQuotient[R, i]]
    }
}

/// A representative whose coset is zero lies in the ideal.
theorem ideal_quotient_contains_of_mk_type_eq_zero[R: CommRing](i: Ideal[R], a: R) {
    ideal_quotient_mk_type(i, a) = Zero.0[IdealQuotient[R, i]] implies i.contains(a)
} by {
    if ideal_quotient_mk_type(i, a) = Zero.0[IdealQuotient[R, i]] {
        ideal_quotient_mk_type_eq_zero_iff_contains(i, a)
        i.contains(a)
    }
}

/// The canonical projection preserves negation.
theorem ideal_quotient_mk_type_neg[R: CommRing](i: Ideal[R], a: R) {
    ideal_quotient_mk_type(i, -a) = -ideal_quotient_mk_type(i, a)
} by {
    ideal_quotient_type_neg_mk(i, a)
    ideal_quotient_type_neg(i, ideal_quotient_mk_type(i, a)) = ideal_quotient_mk_type(i, -a)
    ideal_quotient_typeclass_neg_eq(i, ideal_quotient_mk_type(i, a))
    -ideal_quotient_mk_type(i, a) = ideal_quotient_type_neg(i, ideal_quotient_mk_type(i, a))
    -ideal_quotient_mk_type(i, a) = ideal_quotient_mk_type(i, -a)
}

/// The first isomorphism theorem, factorization half: `f` agrees with the
/// induced map out of `R/ker(f)` on every canonical coset.
theorem ideal_quotient_first_iso_factorization[R: CommRing, S: CommRing](f: RingHom[R, S]) {
    forall(a: R) {
        ring_hom_kernel_quotient_induced(f, ideal_quotient_project(ring_hom_kernel(f), a)) = f.hom(a)
    }
} by {
    forall(a: R) {
        ideal_quotient_kernel_induced_project(f, a)
        ring_hom_kernel_quotient_induced(f, ideal_quotient_project(ring_hom_kernel(f), a)) = f.hom(a)
    }
}

/// The first isomorphism theorem, injectivity half: the induced map out of
/// `R/ker(f)` is injective.
theorem ideal_quotient_first_iso_induced_injective[R: CommRing, S: CommRing](f: RingHom[R, S]) {
    forall(a: R, b: R) {
        ring_hom_kernel_quotient_induced(f, ideal_quotient_project(ring_hom_kernel(f), a)) =
            ring_hom_kernel_quotient_induced(f, ideal_quotient_project(ring_hom_kernel(f), b))
        implies ideal_quotient_project(ring_hom_kernel(f), a) = ideal_quotient_project(ring_hom_kernel(f), b)
    }
} by {
    forall(a: R, b: R) {
        if ring_hom_kernel_quotient_induced(f, ideal_quotient_project(ring_hom_kernel(f), a)) =
            ring_hom_kernel_quotient_induced(f, ideal_quotient_project(ring_hom_kernel(f), b)) {
            ideal_quotient_kernel_induced_injective(f, a, b)
            ideal_quotient_project(ring_hom_kernel(f), a) = ideal_quotient_project(ring_hom_kernel(f), b)
        }
    }
}

// The conjunction `factorization and injectivity` below is the first
// isomorphism theorem.  It is commented out because proof search cannot close
// the top-level conjunction of the two `forall` statements within the 60 s
// timeout (conjunction-intro on goals of this size is not found); the two
// halves above verify individually, and the full first isomorphism theorem is
// already available in `quotient_ring.ac` as `ideal_quotient_kernel_first_iso`,
// `ideal_quotient_kernel_induced_injective`,
// `ideal_quotient_kernel_induced_surjective_onto_image`, and
// `ideal_quotient_kernel_first_iso_induced`.
//
// theorem ideal_quotient_first_isomorphism[R: CommRing, S: CommRing](f: RingHom[R, S]) {
//     (forall(a: R) {
//         ring_hom_kernel_quotient_induced(f, ideal_quotient_project(ring_hom_kernel(f), a)) = f.hom(a)
//     }) and
//     (forall(a: R, b: R) {
//         (ring_hom_kernel_quotient_induced(f, ideal_quotient_project(ring_hom_kernel(f), a)) =
//             ring_hom_kernel_quotient_induced(f, ideal_quotient_project(ring_hom_kernel(f), b))) =
//             (f.hom(a) = f.hom(b))
//     })
// }

// ---------------------------------------------------------------------------
// Section 5: Commutative-ring arithmetic
// ---------------------------------------------------------------------------

/// Multiplication by two, written as `1 + 1`, duplicates an element.
theorem ring_two_mul[R: CommRing](x: R) {
    (R.1 + R.1) * x = x + x
} by {
    (R.1 + R.1) * x = R.1 * x + R.1 * x
    R.1 * x = x
    (R.1 + R.1) * x = x + x
}

/// Expansion of the product `(a + b) * (a + b)` in a commutative ring.
theorem ring_square_add_raw[R: CommRing](a: R, b: R) {
    (a + b) * (a + b) = a * a + (a * b + a * b) + b * b
} by {
    (a + b) * (a + b) = (a + b) * a + (a + b) * b
    (a + b) * a = a * a + b * a
    (a + b) * b = a * b + b * b
    (a + b) * (a + b) = (a * a + b * a) + (a * b + b * b)
    b * a = a * b
    (a + b) * (a + b) = (a * a + a * b) + (a * b + b * b)
    (a * a + a * b) + (a * b + b * b) = a * a + (a * b + (a * b + b * b))
    a * b + (a * b + b * b) = (a * b + a * b) + b * b
    (a * a + a * b) + (a * b + b * b) = a * a + ((a * b + a * b) + b * b)
    a * a + ((a * b + a * b) + b * b) = (a * a + (a * b + a * b)) + b * b
    (a + b) * (a + b) = a * a + (a * b + a * b) + b * b
}

/// Expansion of `(a + b)^2` in a commutative ring.
theorem ring_square_add_pow[R: CommRing](a: R, b: R) {
    (a + b).pow(Nat.2) = a.pow(Nat.2) + (R.1 + R.1) * a * b + b.pow(Nat.2)
} by {
    (a + b).pow(Nat.2) = (a + b) * (a + b)
    ring_square_add_raw[R](a, b)
    (a + b).pow(Nat.2) = a * a + (a * b + a * b) + b * b
    a.pow(Nat.2) = a * a
    b.pow(Nat.2) = b * b
    ring_two_mul[R](a * b)
    (R.1 + R.1) * (a * b) = a * b + a * b
    (R.1 + R.1) * a * b = (R.1 + R.1) * (a * b)
    a * a + (a * b + a * b) + b * b = a.pow(Nat.2) + (R.1 + R.1) * a * b + b.pow(Nat.2)
    (a + b).pow(Nat.2) = a.pow(Nat.2) + (R.1 + R.1) * a * b + b.pow(Nat.2)
}

/// Expansion of `(a - b)^2` in a commutative ring.
theorem ring_square_sub_pow[R: CommRing](a: R, b: R) {
    (a - b).pow(Nat.2) = a.pow(Nat.2) - (R.1 + R.1) * a * b + b.pow(Nat.2)
} by {
    a - b = a + -b
    ring_square_add_pow[R](a, -b)
    (a + -b).pow(Nat.2) = a.pow(Nat.2) + (R.1 + R.1) * a * -b + (-b).pow(Nat.2)
    (a - b).pow(Nat.2) = a.pow(Nat.2) + (R.1 + R.1) * a * -b + (-b).pow(Nat.2)
    mul_neg_right[R]((R.1 + R.1) * a, b)
    (R.1 + R.1) * a * -b = -((R.1 + R.1) * a * b)
    a.pow(Nat.2) - (R.1 + R.1) * a * b = a.pow(Nat.2) + -((R.1 + R.1) * a * b)
    (-b).pow(Nat.2) = (-b) * (-b)
    mul_neg_neg[R](b, b)
    -b * -b = b * b
    b * b = b.pow(Nat.2)
    (-b).pow(Nat.2) = b.pow(Nat.2)
    a.pow(Nat.2) + (R.1 + R.1) * a * -b + (-b).pow(Nat.2) =
        a.pow(Nat.2) - (R.1 + R.1) * a * b + b.pow(Nat.2)
    (a - b).pow(Nat.2) = a.pow(Nat.2) - (R.1 + R.1) * a * b + b.pow(Nat.2)
}

/// Expansion of `(a + b)^2` with the mixed term written last.
theorem ring_square_add_reordered[R: CommRing](a: R, b: R) {
    (a + b).pow(Nat.2) = a.pow(Nat.2) + b.pow(Nat.2) + (R.1 + R.1) * a * b
} by {
    ring_square_add_pow[R](a, b)
    (a + b).pow(Nat.2) = a.pow(Nat.2) + (R.1 + R.1) * a * b + b.pow(Nat.2)
    add_shift_left[R](a.pow(Nat.2), (R.1 + R.1) * a * b, b.pow(Nat.2))
    a.pow(Nat.2) + (R.1 + R.1) * a * b + b.pow(Nat.2) =
        a.pow(Nat.2) + b.pow(Nat.2) + (R.1 + R.1) * a * b
    (a + b).pow(Nat.2) = a.pow(Nat.2) + b.pow(Nat.2) + (R.1 + R.1) * a * b
}

/// The difference of squares in a commutative ring.
theorem ring_difference_of_squares[R: CommRing](a: R, b: R) {
    (a + b) * (a - b) = a.pow(Nat.2) - b.pow(Nat.2)
} by {
    a - b = a + -b
    (a + b) * (a - b) = (a + b) * (a + -b)
    (a + b) * (a + -b) = (a + b) * a + (a + b) * -b
    (a + b) * a = a * a + b * a
    (a + b) * -b = a * -b + b * -b
    mul_neg_right[R](a, b)
    a * -b = -(a * b)
    mul_neg_right[R](b, b)
    b * -b = -(b * b)
    (a + b) * -b = -(a * b) + -(b * b)
    b * a = a * b
    (a + b) * a = a * a + a * b
    (a + b) * (a + -b) = (a * a + a * b) + (-(a * b) + -(b * b))
    (a * a + a * b) + (-(a * b) + -(b * b)) = a * a + (a * b + (-(a * b) + -(b * b)))
    a * b + (-(a * b) + -(b * b)) = (a * b + -(a * b)) + -(b * b)
    a * b + -(a * b) = R.0
    (a * b + -(a * b)) + -(b * b) = R.0 + -(b * b)
    R.0 + -(b * b) = -(b * b)
    (a * a + a * b) + (-(a * b) + -(b * b)) = a * a + -(b * b)
    a.pow(Nat.2) = a * a
    b.pow(Nat.2) = b * b
    a.pow(Nat.2) - b.pow(Nat.2) = a.pow(Nat.2) + -b.pow(Nat.2)
    (a + b) * (a - b) = a.pow(Nat.2) - b.pow(Nat.2)
}

// ---------------------------------------------------------------------------
// Section 6: Additive-commutativity combinations in a ring
// ---------------------------------------------------------------------------

/// Rotating three summands: `(a + b) + c = (b + c) + a`.
theorem ring_add_rotate_three[R: Ring](a: R, b: R, c: R) {
    (a + b) + c = (b + c) + a
} by {
    b + c = c + b
    (c + b) + a = c + (b + a)
    b + a = a + b
    c + (a + b) = (a + b) + c
    (b + c) + a = (a + b) + c
    (a + b) + c = (b + c) + a
}

/// Swapping the first two summands of an associated sum: `a + (b + c) = b + (a + c)`.
theorem ring_add_swap_outer[R: Ring](a: R, b: R, c: R) {
    a + (b + c) = b + (a + c)
} by {
    a + (b + c) = (a + b) + c
    a + b = b + a
    (a + b) + c = (b + a) + c
    (b + a) + c = b + (a + c)
    a + (b + c) = b + (a + c)
}
