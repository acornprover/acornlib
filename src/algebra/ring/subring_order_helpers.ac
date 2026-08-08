from algebra.ring.ring import Ring
from data.basic.set import Set, subset_contains
from algebra.subring import Subring, subring_subset, subring_subset_as_set_eq,
    subring_subset_refl, subring_subset_trans, subring_subset_antisymm,
    subring_intersection_subset_left_relation, subring_intersection_subset_right_relation,
    subring_subset_intersection_of_subset_left_right, subring_subset_intersection_iff,
    subring_closure, subring_closure_mono, subring_closure_le_iff_set_subset,
    set_subset_subring_as_set_eq, subring_sup, subring_subset_sup_left,
    subring_subset_sup_right, subring_sup_subset_of_subset_left_right

/// Subring inclusion is exactly elementwise implication of membership predicates.
theorem subring_subset_contains_eq[R: Ring](a: Subring[R], b: Subring[R]) {
    subring_subset(a, b) = forall(x: R) {
        a.contains(x) implies b.contains(x)
    }
} by {
    subring_subset(a, b) = forall(x: R) {
        a.contains(x) implies b.contains(x)
    }
}

/// A subring inclusion transports membership from the smaller subring to the larger one.
theorem subring_subset_contains[R: Ring](a: Subring[R], b: Subring[R], x: R) {
    subring_subset(a, b) and a.contains(x) implies b.contains(x)
} by {
    if subring_subset(a, b) and a.contains(x) {
        subring_subset_contains_eq(a, b)
        a.contains(x) implies b.contains(x)
        b.contains(x)
    }
}

/// Elementwise membership implication gives subring inclusion.
theorem subring_subset_of_contains[R: Ring](a: Subring[R], b: Subring[R]) {
    (forall(x: R) { a.contains(x) implies b.contains(x) }) implies subring_subset(a, b)
} by {
    if forall(x: R) { a.contains(x) implies b.contains(x) } {
        subring_subset_contains_eq(a, b)
        subring_subset(a, b)
    }
}

/// Subring inclusion is reflexive.
theorem subring_order_subset_refl[R: Ring](a: Subring[R]) {
    subring_subset(a, a)
} by {
    subring_subset_refl(a)
}

/// Subring inclusion is transitive.
theorem subring_order_subset_trans[R: Ring](a: Subring[R], b: Subring[R], c: Subring[R]) {
    subring_subset(a, b) and subring_subset(b, c) implies subring_subset(a, c)
} by {
    if subring_subset(a, b) and subring_subset(b, c) {
        subring_subset_trans(a, b, c)
    }
}

/// Mutual subring inclusion forces equality.
theorem subring_order_subset_antisymm[R: Ring](a: Subring[R], b: Subring[R]) {
    subring_subset(a, b) and subring_subset(b, a) implies a = b
} by {
    if subring_subset(a, b) and subring_subset(b, a) {
        subring_subset_antisymm(a, b)
    }
}

/// Subring inclusion is exactly inclusion of the underlying sets.
theorem subring_subset_as_set_iff[R: Ring](a: Subring[R], b: Subring[R]) {
    subring_subset(a, b) = a.as_set.subset(b.as_set)
} by {
    subring_subset_as_set_eq(a, b)
}

/// Underlying-set inclusion transports subring membership.
theorem subring_as_set_subset_contains[R: Ring](a: Subring[R], b: Subring[R], x: R) {
    a.as_set.subset(b.as_set) and a.contains(x) implies b.contains(x)
} by {
    if a.as_set.subset(b.as_set) and a.contains(x) {
        a.as_set.contains(x)
        subset_contains(a.as_set, b.as_set, x)
        b.as_set.contains(x)
        subring_subset_as_set_eq(a, b)
        subring_subset(a, b)
        subring_subset_contains(a, b, x)
    }
}

/// The intersection is the greatest lower bound for subring inclusion.
theorem subring_intersection_greatest_lower_bound[R: Ring](c: Subring[R], a: Subring[R], b: Subring[R]) {
    subring_subset(c, a.intersection(b)) = (subring_subset(c, a) and subring_subset(c, b))
} by {
    subring_subset_intersection_iff(c, a, b)
}

/// A subring contained in an intersection is contained in the left factor.
theorem subring_subset_left_of_subset_intersection[R: Ring](c: Subring[R], a: Subring[R], b: Subring[R]) {
    subring_subset(c, a.intersection(b)) implies subring_subset(c, a)
} by {
    if subring_subset(c, a.intersection(b)) {
        subring_intersection_subset_left_relation(a, b)
        subring_subset_trans(c, a.intersection(b), a)
        subring_subset(c, a)
    }
}

/// A subring contained in an intersection is contained in the right factor.
theorem subring_subset_right_of_subset_intersection[R: Ring](c: Subring[R], a: Subring[R], b: Subring[R]) {
    subring_subset(c, a.intersection(b)) implies subring_subset(c, b)
} by {
    if subring_subset(c, a.intersection(b)) {
        subring_intersection_subset_right_relation(a, b)
        subring_subset_trans(c, a.intersection(b), b)
        subring_subset(c, b)
    }
}

/// A common subring of two subrings is contained in their intersection.
theorem subring_subset_intersection_of_subset_both[R: Ring](c: Subring[R], a: Subring[R], b: Subring[R]) {
    subring_subset(c, a) and subring_subset(c, b) implies subring_subset(c, a.intersection(b))
} by {
    if subring_subset(c, a) and subring_subset(c, b) {
        subring_subset_intersection_of_subset_left_right(c, a, b)
    }
}

/// Subring closure is monotone with respect to inclusion of generating sets.
theorem subring_closure_monotone[R: Ring](a: Set[R], b: Set[R]) {
    a.subset(b) implies subring_subset(subring_closure(a), subring_closure(b))
} by {
    if a.subset(b) {
        subring_closure_mono(a, b)
    }
}

/// The underlying set of subring closure is monotone with respect to inclusion of generating sets.
theorem subring_closure_as_set_monotone[R: Ring](a: Set[R], b: Set[R]) {
    a.subset(b) implies subring_closure(a).as_set.subset(subring_closure(b).as_set)
} by {
    if a.subset(b) {
        subring_closure_monotone(a, b)
        subring_subset_as_set_eq(subring_closure(a), subring_closure(b))
        subring_closure(a).as_set.subset(subring_closure(b).as_set)
    }
}

/// The closure of a set is contained in a subring exactly when the set is contained in its underlying set.
theorem subring_closure_subset_iff_as_set_subset[R: Ring](a: Set[R], s: Subring[R]) {
    subring_subset(subring_closure(a), s) = a.subset(s.as_set)
} by {
    subring_closure_le_iff_set_subset(a, s)
    set_subset_subring_as_set_eq(a, s)
}

/// Mutual inclusion of subrings is equivalent to equality.
theorem subring_subset_antisymm_iff_eq[R: Ring](a: Subring[R], b: Subring[R]) {
    (subring_subset(a, b) and subring_subset(b, a)) = (a = b)
} by {
    if subring_subset(a, b) and subring_subset(b, a) {
        subring_subset_antisymm(a, b)
        a = b
    }
    if a = b {
        subring_subset_refl(a)
        subring_subset(a, b)
        subring_subset(b, a)
    }
    (subring_subset(a, b) and subring_subset(b, a)) = (a = b)
}

/// Intersections of subrings are monotone in the left argument.
theorem subring_intersection_mono_left[R: Ring](a: Subring[R], b: Subring[R], c: Subring[R]) {
    subring_subset(a, b) implies subring_subset(a.intersection(c), b.intersection(c))
} by {
    if subring_subset(a, b) {
        subring_intersection_subset_left_relation(a, c)
        subring_subset_trans(a.intersection(c), a, b)
        subring_subset(a.intersection(c), b)
        subring_intersection_subset_right_relation(a, c)
        subring_subset(a.intersection(c), c)
        subring_subset_intersection_of_subset_left_right(a.intersection(c), b, c)
        subring_subset(a.intersection(c), b.intersection(c))
    }
}

/// Intersections of subrings are monotone in the right argument.
theorem subring_intersection_mono_right[R: Ring](a: Subring[R], b: Subring[R], c: Subring[R]) {
    subring_subset(a, b) implies subring_subset(c.intersection(a), c.intersection(b))
} by {
    if subring_subset(a, b) {
        subring_intersection_subset_left_relation(c, a)
        subring_subset(c.intersection(a), c)
        subring_intersection_subset_right_relation(c, a)
        subring_subset_trans(c.intersection(a), a, b)
        subring_subset(c.intersection(a), b)
        subring_subset_intersection_of_subset_left_right(c.intersection(a), c, b)
        subring_subset(c.intersection(a), c.intersection(b))
    }
}

/// Intersections of subrings are monotone in both arguments.
theorem subring_intersection_mono[R: Ring](a: Subring[R], b: Subring[R], c: Subring[R], d: Subring[R]) {
    subring_subset(a, b) and subring_subset(c, d) implies
        subring_subset(a.intersection(c), b.intersection(d))
} by {
    if subring_subset(a, b) and subring_subset(c, d) {
        subring_intersection_mono_left(a, b, c)
        subring_subset(a.intersection(c), b.intersection(c))
        subring_intersection_mono_right(c, d, b)
        subring_subset(b.intersection(c), b.intersection(d))
        subring_subset_trans(a.intersection(c), b.intersection(c), b.intersection(d))
        subring_subset(a.intersection(c), b.intersection(d))
    }
}

/// Joins of subrings are monotone in the left argument.
theorem subring_sup_mono_left[R: Ring](a: Subring[R], b: Subring[R], c: Subring[R]) {
    subring_subset(a, b) implies subring_subset(subring_sup(a, c), subring_sup(b, c))
} by {
    if subring_subset(a, b) {
        subring_subset_sup_left(b, c)
        subring_subset_trans(a, b, subring_sup(b, c))
        subring_subset(a, subring_sup(b, c))
        subring_subset_sup_right(b, c)
        subring_subset(c, subring_sup(b, c))
        subring_sup_subset_of_subset_left_right(a, c, subring_sup(b, c))
        subring_subset(subring_sup(a, c), subring_sup(b, c))
    }
}

/// Joins of subrings are monotone in the right argument.
theorem subring_sup_mono_right[R: Ring](a: Subring[R], b: Subring[R], c: Subring[R]) {
    subring_subset(a, b) implies subring_subset(subring_sup(c, a), subring_sup(c, b))
} by {
    if subring_subset(a, b) {
        subring_subset_sup_left(c, b)
        subring_subset(c, subring_sup(c, b))
        subring_subset_sup_right(c, b)
        subring_subset_trans(a, b, subring_sup(c, b))
        subring_subset(a, subring_sup(c, b))
        subring_sup_subset_of_subset_left_right(c, a, subring_sup(c, b))
        subring_subset(subring_sup(c, a), subring_sup(c, b))
    }
}

/// Joins of subrings are monotone in both arguments.
theorem subring_sup_mono[R: Ring](a: Subring[R], b: Subring[R], c: Subring[R], d: Subring[R]) {
    subring_subset(a, b) and subring_subset(c, d) implies
        subring_subset(subring_sup(a, c), subring_sup(b, d))
} by {
    if subring_subset(a, b) and subring_subset(c, d) {
        subring_subset_sup_left(b, d)
        subring_subset_trans(a, b, subring_sup(b, d))
        subring_subset(a, subring_sup(b, d))
        subring_subset_sup_right(b, d)
        subring_subset_trans(c, d, subring_sup(b, d))
        subring_subset(c, subring_sup(b, d))
        subring_sup_subset_of_subset_left_right(a, c, subring_sup(b, d))
        subring_subset(subring_sup(a, c), subring_sup(b, d))
    }
}

/// Mutual containment in generated subrings identifies two subring closures.
theorem subring_closure_eq_of_mutual_as_set_subset[R: Ring](a: Set[R], b: Set[R]) {
    a.subset(subring_closure(b).as_set) and b.subset(subring_closure(a).as_set) implies
        subring_closure(a) = subring_closure(b)
} by {
    if a.subset(subring_closure(b).as_set) and b.subset(subring_closure(a).as_set) {
        subring_closure_subset_iff_as_set_subset(a, subring_closure(b))
        subring_subset(subring_closure(a), subring_closure(b))
        subring_closure_subset_iff_as_set_subset(b, subring_closure(a))
        subring_subset(subring_closure(b), subring_closure(a))
        subring_subset_antisymm(subring_closure(a), subring_closure(b))
        subring_closure(a) = subring_closure(b)
    }
}
