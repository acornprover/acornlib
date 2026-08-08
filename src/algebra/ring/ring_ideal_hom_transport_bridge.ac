/// Compact ideal image/preimage transport bridges for bundled ring homomorphisms.

from comm_ring import CommRing
from algebra.ring.ideal import Ideal, bundled_ideal_subset, bundled_ideal_subset_refl,
    bundled_ideal_subset_trans, ideal_preimage, ring_hom_kernel
from algebra.ring.ring_hom import RingHom, compose_ring_hom
from algebra.ring.ring_ideal_hom_image import ideal_image, ideal_image_subset_iff_subset_preimage
from algebra.ring.ring_ideal_hom_preimage import ring_hom_kernel_compose_eq_preimage

/// Unit of the image/preimage Galois connection: every source ideal is contained
/// in the preimage of its image ideal.
theorem ideal_subset_preimage_image[R: CommRing, S: CommRing](
    f: RingHom[R, S], i: Ideal[R]
) {
    bundled_ideal_subset(i, ideal_preimage(f, ideal_image(f, i)))
} by {
    bundled_ideal_subset_refl(ideal_image(f, i))
    bundled_ideal_subset(ideal_image(f, i), ideal_image(f, i))
    ideal_image_subset_iff_subset_preimage(f, i, ideal_image(f, i))
    bundled_ideal_subset(i, ideal_preimage(f, ideal_image(f, i)))
}

/// Counit of the image/preimage Galois connection: the image of a target ideal's
/// preimage is contained in the target ideal.
theorem ideal_image_preimage_subset[R: CommRing, S: CommRing](
    f: RingHom[R, S], j: Ideal[S]
) {
    bundled_ideal_subset(ideal_image(f, ideal_preimage(f, j)), j)
} by {
    bundled_ideal_subset_refl(ideal_preimage(f, j))
    bundled_ideal_subset(ideal_preimage(f, j), ideal_preimage(f, j))
    ideal_image_subset_iff_subset_preimage(f, ideal_preimage(f, j), j)
    bundled_ideal_subset(ideal_image(f, ideal_preimage(f, j)), j)
}

/// The ideal-image operation is monotone in the source ideal.
theorem ideal_image_mono[R: CommRing, S: CommRing](
    f: RingHom[R, S], i: Ideal[R], j: Ideal[R]
) {
    bundled_ideal_subset(i, j) implies bundled_ideal_subset(ideal_image(f, i), ideal_image(f, j))
} by {
    if bundled_ideal_subset(i, j) {
        ideal_subset_preimage_image(f, j)
        bundled_ideal_subset(j, ideal_preimage(f, ideal_image(f, j)))
        bundled_ideal_subset_trans(i, j, ideal_preimage(f, ideal_image(f, j)))
        bundled_ideal_subset(i, ideal_preimage(f, ideal_image(f, j)))
        ideal_image_subset_iff_subset_preimage(f, i, ideal_image(f, j))
        bundled_ideal_subset(ideal_image(f, i), ideal_image(f, j))
    }
}

/// The image of a source ideal is contained in the zero ideal exactly when the
/// source ideal is contained in the kernel ideal.
theorem ideal_image_subset_zero_iff_subset_kernel[R: CommRing, S: CommRing](
    f: RingHom[R, S], i: Ideal[R]
) {
    bundled_ideal_subset(ideal_image(f, i), Ideal[S].zero) =
        bundled_ideal_subset(i, ring_hom_kernel(f))
} by {
    ideal_image_subset_iff_subset_preimage(f, i, Ideal[S].zero)
    ring_hom_kernel(f) = ideal_preimage(f, Ideal[S].zero)
    bundled_ideal_subset(ideal_image(f, i), Ideal[S].zero) =
        bundled_ideal_subset(i, ring_hom_kernel(f))
}

/// Exactness-style transport across a composite: the image of a source ideal under
/// `g` is contained in the downstream kernel iff the source ideal is contained in
/// the kernel of the composite homomorphism.
theorem ideal_image_subset_kernel_iff_subset_kernel_compose[R: CommRing, S: CommRing, T: CommRing](
    f: RingHom[S, T], g: RingHom[R, S], i: Ideal[R]
) {
    bundled_ideal_subset(ideal_image(g, i), ring_hom_kernel(f)) =
        bundled_ideal_subset(i, ring_hom_kernel(compose_ring_hom(f, g)))
} by {
    ideal_image_subset_iff_subset_preimage(g, i, ring_hom_kernel(f))
    ring_hom_kernel_compose_eq_preimage(f, g)
    ring_hom_kernel(compose_ring_hom(f, g)) = ideal_preimage(g, ring_hom_kernel(f))
    bundled_ideal_subset(ideal_image(g, i), ring_hom_kernel(f)) =
        bundled_ideal_subset(i, ring_hom_kernel(compose_ring_hom(f, g)))
}
