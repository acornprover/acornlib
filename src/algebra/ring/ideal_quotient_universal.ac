/// Universal-property style factorization of ring homomorphisms through ideal quotients.

from comm_ring import CommRing
from algebra.ring.ideal import Ideal, ideal_as_set_contains_eq, ideal_contains_as_set_eq,
    bundled_ideal_subset, bundled_ideal_subset_as_set_eq,
    ring_hom_kernel, ring_hom_maps_to_zero_of_kernel_contains
from algebra.ring.ring_hom import RingHom, ring_hom_add, ring_hom_mul, ring_hom_zero,
    ring_hom_one, ring_hom_neg
from algebra.hom_kernel_injective import ring_hom_eq_iff_sub_zero
from data.basic.set import subset_contains
from data.basic.equivalence import QuotientOver, quotient_respects, quotient_over_mk,
    quotient_over_mk_surjective, quotient_over_lift_to,
    quotient_over_lift_to_projection
from algebra.ring.ideal_quotient import ideal_quotient_relation, ideal_quotient_relation_rel,
    ideal_quotient_project, ideal_quotient_project_eq_mk,
    ideal_quotient_add, ideal_quotient_mul, ideal_quotient_neg,
    ideal_quotient_zero, ideal_quotient_one,
    ideal_quotient_project_add, ideal_quotient_project_mul,
    ideal_quotient_project_neg, ideal_quotient_project_zero,
    ideal_quotient_project_one
from data.basic.quotient_algebra import ideal_quotient_rel, ideal_quotient_rel_eq_contains_sub

/// The condition that an ideal is contained in the kernel of a ring homomorphism.
define ideal_le_ring_hom_kernel[R: CommRing, S: CommRing](
    i: Ideal[R], f: RingHom[R, S]
) -> Bool {
    i.as_set.subset(ring_hom_kernel(f).as_set)
}

/// Kernel containment agrees with bundled ideal containment into the ring-hom kernel.
theorem ideal_le_ring_hom_kernel_iff_bundled_ideal_subset_kernel[R: CommRing, S: CommRing](
    i: Ideal[R], f: RingHom[R, S]
) {
    ideal_le_ring_hom_kernel(i, f) = bundled_ideal_subset(i, ring_hom_kernel(f))
} by {
    let k = ring_hom_kernel(f)
    ideal_le_ring_hom_kernel(i, f) = i.as_set.subset(k.as_set)
    bundled_ideal_subset_as_set_eq(i, k)
    bundled_ideal_subset(i, k) = i.as_set.subset(k.as_set)
    ideal_le_ring_hom_kernel(i, f) = bundled_ideal_subset(i, k)
    ideal_le_ring_hom_kernel(i, f) = bundled_ideal_subset(i, ring_hom_kernel(f))
}

/// Kernel containment unfolds to pointwise membership in the ring-hom kernel.
theorem ideal_le_ring_hom_kernel_contains_kernel[R: CommRing, S: CommRing](
    i: Ideal[R], f: RingHom[R, S], x: R
) {
    ideal_le_ring_hom_kernel(i, f) and i.contains(x) implies ring_hom_kernel(f).contains(x)
} by {
    let k = ring_hom_kernel(f)
    if ideal_le_ring_hom_kernel(i, f) and i.contains(x) {
        ideal_contains_as_set_eq(i, x)
        subset_contains(i.as_set, k.as_set, x)
        k.as_set.contains(x)
        ideal_as_set_contains_eq(k, x)
        ring_hom_kernel(f).contains(x)
    }
}

/// Kernel containment turns ideal membership into vanishing under the homomorphism.
theorem ideal_le_ring_hom_kernel_maps_to_zero[R: CommRing, S: CommRing](
    i: Ideal[R], f: RingHom[R, S], x: R
) {
    ideal_le_ring_hom_kernel(i, f) and i.contains(x) implies f.hom(x) = S.0
} by {
    if ideal_le_ring_hom_kernel(i, f) and i.contains(x) {
        ideal_le_ring_hom_kernel_contains_kernel(i, f, x)
        ring_hom_maps_to_zero_of_kernel_contains(f, x)
        f.hom(x) = S.0
    }
}

/// If an ideal is contained in the kernel, the homomorphism is constant on
/// ideal-quotient-related representatives.
theorem ideal_quotient_rel_respected_by_hom[R: CommRing, S: CommRing](
    i: Ideal[R], f: RingHom[R, S], a: R, b: R
) {
    ideal_le_ring_hom_kernel(i, f) and ideal_quotient_rel(i, a, b)
        implies f.hom(a) = f.hom(b)
} by {
    if ideal_le_ring_hom_kernel(i, f) and ideal_quotient_rel(i, a, b) {
        ideal_quotient_rel_eq_contains_sub(i, a, b)
        ideal_le_ring_hom_kernel_maps_to_zero(i, f, a - b)
        ring_hom_eq_iff_sub_zero(f, b, a)
        f.hom(a) = f.hom(b)
    }
}

/// Kernel containment says the representative map respects the ideal quotient relation.
theorem ideal_quotient_lift_respects[R: CommRing, S: CommRing](
    i: Ideal[R], f: RingHom[R, S]
) {
    ideal_le_ring_hom_kernel(i, f) implies quotient_respects(ideal_quotient_relation(i), f.hom)
} by {
    if ideal_le_ring_hom_kernel(i, f) {
        forall(a: R, b: R) {
            if ideal_quotient_relation(i).rel(a, b) {
                ideal_quotient_relation_rel(i)
                ideal_quotient_rel_respected_by_hom(i, f, a, b)
                f.hom(a) = f.hom(b)
            }
        }
    }
}

/// The map out of `R/I` induced by a homomorphism whose kernel contains `I`.
define ideal_quotient_lift[R: CommRing, S: CommRing](
    i: Ideal[R], f: RingHom[R, S]
) -> QuotientOver[R] -> S {
    quotient_over_lift_to(f.hom)
}

/// The induced map agrees with the original homomorphism after quotient projection.
theorem ideal_quotient_lift_project[R: CommRing, S: CommRing](
    i: Ideal[R], f: RingHom[R, S], a: R
) {
    ideal_le_ring_hom_kernel(i, f) implies
    ideal_quotient_lift(i, f, ideal_quotient_project(i, a)) = f.hom(a)
} by {
    if ideal_le_ring_hom_kernel(i, f) {
        ideal_quotient_lift_respects(i, f)
        ideal_quotient_project_eq_mk(i, a)
        quotient_over_lift_to_projection(ideal_quotient_relation(i), f.hom, a)
        ideal_quotient_lift(i, f, ideal_quotient_project(i, a)) = f.hom(a)
    }
}

/// Agreement of a quotient map with a representative homomorphism at one projection.
define ideal_quotient_lift_agrees_on_project_at[R: CommRing, S: CommRing](
    i: Ideal[R], f: RingHom[R, S], g: QuotientOver[R] -> S, a: R
) -> Bool {
    g(ideal_quotient_project(i, a)) = f.hom(a)
}

/// A quotient map agrees with a representative homomorphism after quotient projection.
define ideal_quotient_lift_agrees_on_projects[R: CommRing, S: CommRing](
    i: Ideal[R], f: RingHom[R, S], g: QuotientOver[R] -> S
) -> Bool {
    forall(a: R) { ideal_quotient_lift_agrees_on_project_at(i, f, g, a) }
}

/// A quotient map agreeing with `f` on all projections agrees with the induced
/// lift on every element of this ideal quotient.
theorem ideal_quotient_lift_unique_on_quotient[R: CommRing, S: CommRing](
    i: Ideal[R], f: RingHom[R, S], g: QuotientOver[R] -> S, q: QuotientOver[R]
) {
    ideal_le_ring_hom_kernel(i, f) and q.qrel = ideal_quotient_relation(i) and
    ideal_quotient_lift_agrees_on_projects(i, f, g) implies
    g(q) = ideal_quotient_lift(i, f, q)
} by {
    if ideal_le_ring_hom_kernel(i, f) {
        if q.qrel = ideal_quotient_relation(i) {
            if ideal_quotient_lift_agrees_on_projects(i, f, g) {
                quotient_over_mk_surjective(q)
                let a: R satisfy {
                    quotient_over_mk(q.qrel, a) = q
                }
                ideal_quotient_project_eq_mk(i, a)
                ideal_quotient_lift_agrees_on_project_at(i, f, g, a)
                g(ideal_quotient_project(i, a)) = f.hom(a)
                ideal_quotient_lift_project(i, f, a)
                g(q) = ideal_quotient_lift(i, f, q)
            }
        }
    }
}

/// The induced lift preserves addition on projected representatives.
theorem ideal_quotient_lift_add[R: CommRing, S: CommRing](
    i: Ideal[R], f: RingHom[R, S], a: R, b: R
) {
    ideal_le_ring_hom_kernel(i, f) implies
    ideal_quotient_lift(i, f,
        ideal_quotient_add(i, ideal_quotient_project(i, a), ideal_quotient_project(i, b))) =
    ideal_quotient_lift(i, f, ideal_quotient_project(i, a)) +
    ideal_quotient_lift(i, f, ideal_quotient_project(i, b))
} by {
    if ideal_le_ring_hom_kernel(i, f) {
        ideal_quotient_project_add(i, a, b)
        ideal_quotient_lift_project(i, f, a + b)
        ideal_quotient_lift_project(i, f, a)
        ideal_quotient_lift_project(i, f, b)
        ring_hom_add(f, a, b)
        ideal_quotient_lift(i, f,
            ideal_quotient_add(i, ideal_quotient_project(i, a), ideal_quotient_project(i, b))) =
        ideal_quotient_lift(i, f, ideal_quotient_project(i, a)) +
        ideal_quotient_lift(i, f, ideal_quotient_project(i, b))
    }
}

/// The induced lift preserves multiplication on projected representatives.
theorem ideal_quotient_lift_mul[R: CommRing, S: CommRing](
    i: Ideal[R], f: RingHom[R, S], a: R, b: R
) {
    ideal_le_ring_hom_kernel(i, f) implies
    ideal_quotient_lift(i, f,
        ideal_quotient_mul(i, ideal_quotient_project(i, a), ideal_quotient_project(i, b))) =
    ideal_quotient_lift(i, f, ideal_quotient_project(i, a)) *
    ideal_quotient_lift(i, f, ideal_quotient_project(i, b))
} by {
    if ideal_le_ring_hom_kernel(i, f) {
        ideal_quotient_project_mul(i, a, b)
        ideal_quotient_lift_project(i, f, a * b)
        ideal_quotient_lift_project(i, f, a)
        ideal_quotient_lift_project(i, f, b)
        ring_hom_mul(f, a, b)
        ideal_quotient_lift(i, f,
            ideal_quotient_mul(i, ideal_quotient_project(i, a), ideal_quotient_project(i, b))) =
        ideal_quotient_lift(i, f, ideal_quotient_project(i, a)) *
        ideal_quotient_lift(i, f, ideal_quotient_project(i, b))
    }
}

/// The induced lift preserves quotient zero.
theorem ideal_quotient_lift_zero[R: CommRing, S: CommRing](
    i: Ideal[R], f: RingHom[R, S]
) {
    ideal_le_ring_hom_kernel(i, f) implies
    ideal_quotient_lift(i, f, ideal_quotient_zero(i)) = S.0
} by {
    if ideal_le_ring_hom_kernel(i, f) {
        ideal_quotient_project_zero(i)
        ideal_quotient_lift_project(i, f, R.0)
        ring_hom_zero(f)
        ideal_quotient_lift(i, f, ideal_quotient_zero(i)) = S.0
    }
}

/// The induced lift preserves quotient one.
theorem ideal_quotient_lift_one[R: CommRing, S: CommRing](
    i: Ideal[R], f: RingHom[R, S]
) {
    ideal_le_ring_hom_kernel(i, f) implies
    ideal_quotient_lift(i, f, ideal_quotient_one(i)) = S.1
} by {
    if ideal_le_ring_hom_kernel(i, f) {
        ideal_quotient_project_one(i)
        ideal_quotient_lift_project(i, f, R.1)
        ring_hom_one(f)
        ideal_quotient_lift(i, f, ideal_quotient_one(i)) = S.1
    }
}

/// The induced lift preserves negation on projected representatives.
theorem ideal_quotient_lift_neg[R: CommRing, S: CommRing](
    i: Ideal[R], f: RingHom[R, S], a: R
) {
    ideal_le_ring_hom_kernel(i, f) implies
    ideal_quotient_lift(i, f,
        ideal_quotient_neg(i, ideal_quotient_project(i, a))) =
    -ideal_quotient_lift(i, f, ideal_quotient_project(i, a))
} by {
    if ideal_le_ring_hom_kernel(i, f) {
        ideal_quotient_project_neg(i, a)
        ideal_quotient_lift_project(i, f, -a)
        ideal_quotient_lift_project(i, f, a)
        ring_hom_neg(f, a)
        ideal_quotient_lift(i, f,
            ideal_quotient_neg(i, ideal_quotient_project(i, a))) =
        -ideal_quotient_lift(i, f, ideal_quotient_project(i, a))
    }
}
