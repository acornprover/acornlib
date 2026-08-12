// Characteristic and nilpotence deepening: nilpotent elements of a ring and
// their closure properties, the characteristic-two condition stated as an
// iff on the doubling of one, and the element-level identities equivalent to
// `a + a = 0`.

from comm_ring import CommRing
from algebra.ring.ring import Ring, mul_zero_left, mul_zero_right
from algebra.ring.ring_axioms_deep import ring_pow_two, ring_neg_pow_two, ring_sub_neg,
    ring_add_eq_zero_iff_eq_neg
from nat import Nat, pow_add, pow_pow, pow_zero, pow_one, alt_induction, mul_comm,
    pow_distrib_mul, semiring_zero_pow, alt_suc_ne_zero
numerals Nat

/// True if an element of a ring is nilpotent: some positive power is zero.
define is_nilpotent[R: Ring](a: R) -> Bool {
    exists(n: Nat) {
        n != Nat.0 and a.pow(n) = R.0
    }
}

/// The zero element is nilpotent.
theorem nilpotent_zero[R: Ring] {
    is_nilpotent[R](R.0)
} by {
    is_nilpotent[R](R.0) = exists(n: Nat) {
        n != Nat.0 and R.0.pow(n) = R.0
    }
    R.0.pow(Nat.1) = R.0 * R.0.pow(Nat.0)
    pow_zero(R.0)
    R.0.pow(Nat.0) = R.1
    R.0.pow(Nat.1) = R.0 * R.1
    R.0 * R.1 = R.0
    R.0.pow(Nat.1) = R.0
    Nat.1 != Nat.0
    exists(n: Nat) { n != Nat.0 and R.0.pow(n) = R.0 }
    is_nilpotent[R](R.0)
}

/// A positive power equal to zero witnesses nilpotence.
theorem nilpotent_intro[R: Ring](a: R, n: Nat) {
    n != Nat.0 and a.pow(n) = R.0 implies is_nilpotent[R](a)
} by {
    if n != Nat.0 and a.pow(n) = R.0 {
        is_nilpotent[R](a) = exists(m: Nat) {
            m != Nat.0 and a.pow(m) = R.0
        }
        exists(m: Nat) { m != Nat.0 and a.pow(m) = R.0 }
        is_nilpotent[R](a)
    }
}

/// A nilpotent element has a positive power equal to zero.
theorem nilpotent_apply[R: Ring](a: R) {
    is_nilpotent[R](a) implies exists(n: Nat) {
        n != Nat.0 and a.pow(n) = R.0
    }
} by {
    if is_nilpotent[R](a) {
        is_nilpotent[R](a) = exists(n: Nat) {
            n != Nat.0 and a.pow(n) = R.0
        }
        exists(n: Nat) { n != Nat.0 and a.pow(n) = R.0 }
    }
}

/// The negation of a nilpotent element is nilpotent.
theorem nilpotent_neg[R: CommRing](a: R) {
    is_nilpotent[R](a) implies is_nilpotent[R](-a)
} by {
    if is_nilpotent[R](a) {
        is_nilpotent[R](a) = exists(n: Nat) {
            n != Nat.0 and a.pow(n) = R.0
        }
        let n: Nat satisfy { n != Nat.0 and a.pow(n) = R.0 }
        ring_neg_pow_two(a)
        (-a).pow(2) = a.pow(2)
        pow_pow(-a, 2, n)
        (-a).pow(2).pow(n) = (-a).pow(2 * n)
        pow_pow(a, 2, n)
        a.pow(2).pow(n) = a.pow(2 * n)
        (-a).pow(2 * n) = a.pow(2 * n)
        pow_pow(a, n, 2)
        a.pow(n).pow(2) = a.pow(n * 2)
        ring_pow_two(a.pow(n))
        a.pow(n).pow(2) = a.pow(n) * a.pow(n)
        a.pow(n) = R.0
        a.pow(n) * a.pow(n) = R.0 * a.pow(n)
        mul_zero_left(a.pow(n))
        R.0 * a.pow(n) = R.0
        a.pow(n).pow(2) = R.0
        a.pow(n * 2) = R.0
        mul_comm(n, 2)
        n * 2 = 2 * n
        a.pow(n * 2) = a.pow(2 * n)
        a.pow(2 * n) = R.0
        (-a).pow(2 * n) = R.0
        2 * n != Nat.0
        exists(m: Nat) { m != Nat.0 and (-a).pow(m) = R.0 }
        is_nilpotent[R](-a) = exists(m: Nat) {
            m != Nat.0 and (-a).pow(m) = R.0
        }
        is_nilpotent[R](-a)
    }
}

/// A nilpotent element times any element is nilpotent.
theorem nilpotent_mul_left[R: CommRing](a: R, b: R) {
    is_nilpotent[R](a) implies is_nilpotent[R](a * b)
} by {
    if is_nilpotent[R](a) {
        is_nilpotent[R](a) = exists(n: Nat) {
            n != Nat.0 and a.pow(n) = R.0
        }
        let n: Nat satisfy { n != Nat.0 and a.pow(n) = R.0 }
        pow_distrib_mul(a, b, n)
        (a * b).pow(n) = a.pow(n) * b.pow(n)
        a.pow(n) = R.0
        a.pow(n) * b.pow(n) = R.0 * b.pow(n)
        mul_zero_left(b.pow(n))
        R.0 * b.pow(n) = R.0
        (a * b).pow(n) = R.0
        exists(m: Nat) { m != Nat.0 and (a * b).pow(m) = R.0 }
        is_nilpotent[R](a * b) = exists(m: Nat) {
            m != Nat.0 and (a * b).pow(m) = R.0
        }
        is_nilpotent[R](a * b)
    }
}

/// Any element times a nilpotent element is nilpotent.
theorem nilpotent_mul_right[R: CommRing](a: R, b: R) {
    is_nilpotent[R](b) implies is_nilpotent[R](a * b)
} by {
    if is_nilpotent[R](b) {
        is_nilpotent[R](b) = exists(n: Nat) {
            n != Nat.0 and b.pow(n) = R.0
        }
        let n: Nat satisfy { n != Nat.0 and b.pow(n) = R.0 }
        pow_distrib_mul(a, b, n)
        (a * b).pow(n) = a.pow(n) * b.pow(n)
        b.pow(n) = R.0
        a.pow(n) * b.pow(n) = a.pow(n) * R.0
        mul_zero_right(a.pow(n))
        a.pow(n) * R.0 = R.0
        (a * b).pow(n) = R.0
        exists(m: Nat) { m != Nat.0 and (a * b).pow(m) = R.0 }
        is_nilpotent[R](a * b) = exists(m: Nat) {
            m != Nat.0 and (a * b).pow(m) = R.0
        }
        is_nilpotent[R](a * b)
    }
}

/// A positive power of a nilpotent element is nilpotent.
theorem nilpotent_pow[R: Ring](a: R, k: Nat) {
    k != Nat.0 and is_nilpotent[R](a) implies is_nilpotent[R](a.pow(k))
} by {
    if k != Nat.0 and is_nilpotent[R](a) {
        is_nilpotent[R](a) = exists(n: Nat) {
            n != Nat.0 and a.pow(n) = R.0
        }
        let n: Nat satisfy { n != Nat.0 and a.pow(n) = R.0 }
        pow_pow(a, k, n)
        a.pow(k).pow(n) = a.pow(k * n)
        pow_pow(a, n, k)
        a.pow(n).pow(k) = a.pow(n * k)
        a.pow(n) = R.0
        a.pow(n).pow(k) = R.0.pow(k)
        semiring_zero_pow[R](k)
        k != Nat.0 implies R.0.pow(k) = R.0
        R.0.pow(k) = R.0
        a.pow(n).pow(k) = R.0
        mul_comm(n, k)
        n * k = k * n
        a.pow(n * k) = a.pow(k * n)
        a.pow(k * n) = R.0
        a.pow(k).pow(n) = R.0
        exists(m: Nat) { m != Nat.0 and a.pow(k).pow(m) = R.0 }
        is_nilpotent[R](a.pow(k)) = exists(m: Nat) {
            m != Nat.0 and a.pow(k).pow(m) = R.0
        }
        is_nilpotent[R](a.pow(k))
    }
}

/// True if the ring has characteristic two: every element doubles to zero.
let is_char_two[R: Ring] = forall(a: R) {
    a + a = R.0
}

/// A ring has characteristic two exactly when one plus one is zero.
theorem char_two_iff_one_plus_one_zero[R: Ring] {
    is_char_two[R] = (R.1 + R.1 = R.0)
} by {
    if is_char_two[R] {
        is_char_two[R] = forall(a: R) { a + a = R.0 }
        R.1 + R.1 = R.0
    }
    if R.1 + R.1 = R.0 {
        forall(a: R) {
            (R.1 + R.1) * a = a + a
            (R.1 + R.1) * a = R.0 * a
            R.0 * a = R.0
            a + a = R.0
        }
        is_char_two[R] = forall(a: R) { a + a = R.0 }
        is_char_two[R]
    }
    is_char_two[R] = (R.1 + R.1 = R.0)
}

/// In characteristic two, every element is its own negation.
theorem char_two_neg_eq_self[R: Ring](a: R) {
    is_char_two[R] implies -a = a
} by {
    if is_char_two[R] {
        is_char_two[R] = forall(x: R) { x + x = R.0 }
        a + a = R.0
        ring_add_eq_zero_iff_eq_neg(a, a)
        (a + a = R.0) = (a = -a)
        a = -a
        -a = a
    }
}

/// In characteristic two, subtraction agrees with addition.
theorem char_two_sub_eq_add[R: Ring](a: R, b: R) {
    is_char_two[R] implies a - b = a + b
} by {
    if is_char_two[R] {
        char_two_neg_eq_self(b)
        -b = b
        a - b = a + -b
        a + -b = a + b
        a - b = a + b
    }
}

/// An element doubles to zero exactly when it is its own negation.
theorem ring_add_self_zero_iff_neg_eq_self[R: Ring](a: R) {
    (a + a = R.0) = (-a = a)
} by {
    if a + a = R.0 {
        ring_add_eq_zero_iff_eq_neg(a, a)
        (a + a = R.0) = (a = -a)
        a = -a
        -a = a
    }
    if -a = a {
        a + a = a + -a
        a + -a = R.0
        a + a = R.0
    }
    (a + a = R.0) = (-a = a)
}

/// An element doubles to zero exactly when the doubled multiplication annihilates it.
theorem ring_add_self_zero_iff_two_mul_zero[R: Ring](a: R) {
    (a + a = R.0) = ((R.1 + R.1) * a = R.0)
} by {
    if a + a = R.0 {
        (R.1 + R.1) * a = R.1 * a + R.1 * a
        R.1 * a = a
        (R.1 + R.1) * a = a + a
        a + a = R.0
        (R.1 + R.1) * a = R.0
    }
    if (R.1 + R.1) * a = R.0 {
        (R.1 + R.1) * a = R.1 * a + R.1 * a
        R.1 * a = a
        (R.1 + R.1) * a = a + a
        a + a = R.0
    }
    (a + a = R.0) = ((R.1 + R.1) * a = R.0)
}
