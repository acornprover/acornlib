from algebra.ring.ring import Ring
from algebra.semiring_hom import SemiringHom, semiring_hom_one, semiring_hom_add, semiring_hom_mul,
    semiring_hom_eq_of_hom_eq
from algebra.ring.ring_hom import RingHom, is_ring_hom, ring_hom_neg, ring_hom_eq_of_hom_eq,
    ring_hom_to_semiring_hom, ring_hom_to_semiring_hom_hom

/// A semiring homomorphism between rings preserves the data required of a ring homomorphism.
theorem semiring_hom_is_ring_hom[R: Ring, S: Ring](f: SemiringHom[R, S]) {
    is_ring_hom(f.hom)
} by {
    semiring_hom_one(f)
    f.hom(R.1) = S.1
    forall(a: R, b: R) {
        semiring_hom_add(f, a, b)
        semiring_hom_mul(f, a, b)
        f.hom(a + b) = f.hom(a) + f.hom(b)
        f.hom(a * b) = f.hom(a) * f.hom(b)
        f.hom(a + b) = f.hom(a) + f.hom(b) and f.hom(a * b) = f.hom(a) * f.hom(b)
    }
    f.hom(R.1) = S.1 and forall(a: R, b: R) {
        f.hom(a + b) = f.hom(a) + f.hom(b) and f.hom(a * b) = f.hom(a) * f.hom(b)
    }
    is_ring_hom(f.hom)
}

/// Convert a bundled semiring homomorphism between rings into a bundled ring homomorphism.
let semiring_hom_to_ring_hom[R: Ring, S: Ring](f: SemiringHom[R, S]) -> result: RingHom[R, S] satisfy {
    RingHom.new(f.hom) = Option.some(result)
} by {
    semiring_hom_is_ring_hom(f)
}

/// The ring homomorphism obtained from a semiring homomorphism has the same underlying function.
theorem semiring_hom_to_ring_hom_hom[R: Ring, S: Ring](f: SemiringHom[R, S]) {
    semiring_hom_to_ring_hom(f).hom = f.hom
}

/// Converting a ring homomorphism to a semiring homomorphism and back is the identity.
theorem ring_to_semiring_to_ring_hom[R: Ring, S: Ring](f: RingHom[R, S]) {
    semiring_hom_to_ring_hom(ring_hom_to_semiring_hom(f)) = f
} by {
    let g = ring_hom_to_semiring_hom(f)
    semiring_hom_to_ring_hom_hom(g)
    ring_hom_to_semiring_hom_hom(f)
    semiring_hom_to_ring_hom(g).hom = g.hom
    g.hom = f.hom
    semiring_hom_to_ring_hom(g).hom = f.hom
    ring_hom_eq_of_hom_eq(semiring_hom_to_ring_hom(g), f)
}

/// Converting a semiring homomorphism between rings to a ring homomorphism and back is the identity.
theorem semiring_to_ring_to_semiring_hom[R: Ring, S: Ring](f: SemiringHom[R, S]) {
    ring_hom_to_semiring_hom(semiring_hom_to_ring_hom(f)) = f
} by {
    let g = semiring_hom_to_ring_hom(f)
    ring_hom_to_semiring_hom_hom(g)
    semiring_hom_to_ring_hom_hom(f)
    ring_hom_to_semiring_hom(g).hom = g.hom
    g.hom = f.hom
    ring_hom_to_semiring_hom(g).hom = f.hom
    semiring_hom_eq_of_hom_eq(ring_hom_to_semiring_hom(g), f)
}

/// A semiring homomorphism between rings preserves additive inverses.
theorem semiring_hom_ring_neg[R: Ring, S: Ring](f: SemiringHom[R, S], a: R) {
    f.hom(-a) = -f.hom(a)
} by {
    let g = semiring_hom_to_ring_hom(f)
    semiring_hom_to_ring_hom_hom(f)
    g.hom = f.hom
    ring_hom_neg(g, a)
    g.hom(-a) = -g.hom(a)
    f.hom(-a) = g.hom(-a)
    g.hom(a) = f.hom(a)
    -g.hom(a) = -f.hom(a)
    f.hom(-a) = -f.hom(a)
}
