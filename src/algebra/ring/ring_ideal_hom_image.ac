/// Ideal images and the image-preimage Galois bridge for ring homomorphisms.

from comm_ring import CommRing
from algebra.ring.ring_hom import RingHom
from data.basic.relation_transport import predicate_subset, predicate_subset_step
from data.basic.set import set_image, maps_into_set_image, set_image_contains_witness
from algebra.ring.ideal import Ideal, bundled_ideal_subset, ideal_as_set_contains_eq, set_subset_ideal,
    ideal_closure_contains_of_set_contains, ideal_closure_subset_of_set_subset,
    ideal_preimage, ideal_preimage_contains_eq, ideal_contains_map_of_preimage_contains

/// Bundled ideal containment transports membership of a single element.
theorem bundled_ideal_subset_contains_at[R: CommRing](a: Ideal[R], b: Ideal[R], x: R) {
    bundled_ideal_subset(a, b) and a.contains(x) implies b.contains(x)
} by {
    if bundled_ideal_subset(a, b) and a.contains(x) {
        bundled_ideal_subset(a, b) = predicate_subset(a.contains, b.contains)
        predicate_subset(a.contains, b.contains)
        predicate_subset_step(a.contains, b.contains, x)
        b.contains(x)
    }
}

/// The image ideal is the ideal generated by the pointwise image of the underlying set.
define ideal_image[R: CommRing, S: CommRing](f: RingHom[R, S], i: Ideal[R]) -> Ideal[S] {
    Ideal[S].closure(set_image(i.as_set, f.hom))
}

/// The image ideal is the closure of the pointwise image of the underlying set.
theorem ideal_image_eq_closure[R: CommRing, S: CommRing](f: RingHom[R, S], i: Ideal[R]) {
    ideal_image(f, i) = Ideal[S].closure(set_image(i.as_set, f.hom))
} by {
    ideal_image(f, i) = Ideal[S].closure(set_image(i.as_set, f.hom))
}

/// An element of the source ideal maps into the image ideal.
theorem ideal_image_contains_hom[R: CommRing, S: CommRing](f: RingHom[R, S], i: Ideal[R], x: R) {
    i.contains(x) implies ideal_image(f, i).contains(f.hom(x))
} by {
    if i.contains(x) {
        ideal_as_set_contains_eq(i, x)
        i.as_set.contains(x)
        maps_into_set_image(i.as_set, f.hom, x)
        set_image(i.as_set, f.hom).contains(f.hom(x))
        ideal_closure_contains_of_set_contains(set_image(i.as_set, f.hom), f.hom(x))
        Ideal[S].closure(set_image(i.as_set, f.hom)).contains(f.hom(x))
        ideal_image(f, i).contains(f.hom(x))
    }
}

/// Image-ideal containment gives source containment in the inverse-image ideal.
theorem ideal_subset_preimage_of_image_subset[R: CommRing, S: CommRing](
    f: RingHom[R, S], i: Ideal[R], j: Ideal[S]
) {
    bundled_ideal_subset(ideal_image(f, i), j) implies bundled_ideal_subset(i, ideal_preimage(f, j))
} by {
    if bundled_ideal_subset(ideal_image(f, i), j) {
        forall(x: R) {
            if i.contains(x) {
                ideal_image_contains_hom(f, i, x)
                ideal_image(f, i).contains(f.hom(x))
                bundled_ideal_subset_contains_at(ideal_image(f, i), j, f.hom(x))
                j.contains(f.hom(x))
                ideal_preimage_contains_eq(f, j, x)
                ideal_preimage(f, j).contains(x)
            }
        }
    }
}

/// Source containment in the inverse-image ideal gives image-ideal containment.
theorem ideal_image_subset_of_subset_preimage[R: CommRing, S: CommRing](
    f: RingHom[R, S], i: Ideal[R], j: Ideal[S]
) {
    bundled_ideal_subset(i, ideal_preimage(f, j)) implies bundled_ideal_subset(ideal_image(f, i), j)
} by {
    if bundled_ideal_subset(i, ideal_preimage(f, j)) {
        forall(y: S) {
            if set_image(i.as_set, f.hom).contains(y) {
                set_image_contains_witness(i.as_set, f.hom, y)
                let x: R satisfy {
                    i.as_set.contains(x) and y = f.hom(x)
                }
                ideal_as_set_contains_eq(i, x)
                i.contains(x)
                bundled_ideal_subset_contains_at(i, ideal_preimage(f, j), x)
                ideal_preimage(f, j).contains(x)
                ideal_contains_map_of_preimage_contains(f, j, x)
                j.contains(f.hom(x))
                j.contains(y)
            }
        }
        set_subset_ideal(set_image(i.as_set, f.hom), j)
        ideal_closure_subset_of_set_subset(set_image(i.as_set, f.hom), j)
        bundled_ideal_subset(Ideal[S].closure(set_image(i.as_set, f.hom)), j)
        ideal_image(f, i) = Ideal[S].closure(set_image(i.as_set, f.hom))
        bundled_ideal_subset(ideal_image(f, i), j)
    }
}

/// Image and inverse image of ideals form a containment Galois connection.
theorem ideal_image_subset_iff_subset_preimage[R: CommRing, S: CommRing](
    f: RingHom[R, S], i: Ideal[R], j: Ideal[S]
) {
    bundled_ideal_subset(ideal_image(f, i), j) = bundled_ideal_subset(i, ideal_preimage(f, j))
} by {
    if bundled_ideal_subset(ideal_image(f, i), j) {
        ideal_subset_preimage_of_image_subset(f, i, j)
        bundled_ideal_subset(i, ideal_preimage(f, j))
    }
    if bundled_ideal_subset(i, ideal_preimage(f, j)) {
        ideal_image_subset_of_subset_preimage(f, i, j)
        bundled_ideal_subset(ideal_image(f, i), j)
    }
    bundled_ideal_subset(ideal_image(f, i), j) = bundled_ideal_subset(i, ideal_preimage(f, j))
}
