from algebra.ring.ring import Ring
from algebra.ring.ring_hom import RingHom, compose_ring_hom, compose_ring_hom_hom
from data.basic.functions import compose
from data.basic.set import Set, set_image, set_preimage, maps_into_set_image, set_image_contains_witness,
    set_image_subset_iff_subset_preimage, set_image_compose, set_preimage_compose,
    set_image_preimage_closure, set_image_preimage_closure_extensive,
    set_image_preimage_closure_monotone, set_image_preimage_closure_idempotent,
    set_preimage_image_kernel, set_preimage_image_kernel_monotone,
    set_preimage_image_kernel_reductive, set_preimage_image_kernel_idempotent,
    is_subset_monotone_map, is_set_extensive_map, is_set_reductive_map,
    is_set_idempotent_map, is_set_closure_operator, is_set_kernel_operator

/// The image of a set under a ring homomorphism.
define ring_hom_image[S: Ring, T: Ring](f: RingHom[S, T], s: Set[S]) -> Set[T] {
    set_image(s, f.hom)
}

/// The preimage of a set under a ring homomorphism.
define ring_hom_preimage[S: Ring, T: Ring](f: RingHom[S, T], s: Set[T]) -> Set[S] {
    set_preimage(f.hom, s)
}

/// The source closure induced by image followed by preimage along a ring homomorphism.
define ring_hom_image_preimage_closure[S: Ring, T: Ring](
    f: RingHom[S, T],
    s: Set[S]
) -> Set[S] {
    set_image_preimage_closure(f.hom, s)
}

/// The target kernel induced by preimage followed by image along a ring homomorphism.
define ring_hom_preimage_image_kernel[S: Ring, T: Ring](
    f: RingHom[S, T],
    s: Set[T]
) -> Set[T] {
    set_preimage_image_kernel(f.hom, s)
}

/// Membership in the image of a ring homomorphism has a source witness.
theorem ring_hom_image_contains_witness[S: Ring, T: Ring](
    f: RingHom[S, T],
    s: Set[S],
    y: T
) {
    ring_hom_image(f, s).contains(y) implies exists(x: S) {
        s.contains(x) and y = f.hom(x)
    }
} by {
    ring_hom_image(f, s) = set_image(s, f.hom)
    set_image_contains_witness(s, f.hom, y)
}

/// A member of a set maps into its image under a ring homomorphism.
theorem ring_hom_maps_into_image[S: Ring, T: Ring](
    f: RingHom[S, T],
    s: Set[S],
    x: S
) {
    s.contains(x) implies ring_hom_image(f, s).contains(f.hom(x))
} by {
    if s.contains(x) {
        maps_into_set_image(s, f.hom, x)
        set_image(s, f.hom).contains(f.hom(x))
        ring_hom_image(f, s) = set_image(s, f.hom)
        ring_hom_image(f, s).contains(f.hom(x))
    }
}

/// Image containment is equivalent to source containment in the preimage.
theorem ring_hom_image_subset_iff_subset_preimage[S: Ring, T: Ring](
    f: RingHom[S, T],
    s: Set[S],
    u: Set[T]
) {
    ring_hom_image(f, s).subset(u) = s.subset(ring_hom_preimage(f, u))
} by {
    ring_hom_image(f, s) = set_image(s, f.hom)
    ring_hom_preimage(f, u) = set_preimage(f.hom, u)
    set_image_subset_iff_subset_preimage(s, u, f.hom)
}

/// Image under a composite ring homomorphism is iterated image.
theorem ring_hom_image_compose[S: Ring, T: Ring, U: Ring](
    f: RingHom[T, U],
    g: RingHom[S, T],
    s: Set[S]
) {
    ring_hom_image(compose_ring_hom(f, g), s) =
        ring_hom_image(f, ring_hom_image(g, s))
} by {
    compose_ring_hom_hom(f, g)
    ring_hom_image(compose_ring_hom(f, g), s) =
        set_image(s, compose(f.hom, g.hom))
    ring_hom_image(g, s) = set_image(s, g.hom)
    ring_hom_image(f, ring_hom_image(g, s)) =
        set_image(set_image(s, g.hom), f.hom)
    set_image_compose(s, g.hom, f.hom)
}

/// Preimage under a composite ring homomorphism is iterated preimage.
theorem ring_hom_preimage_compose[S: Ring, T: Ring, U: Ring](
    f: RingHom[T, U],
    g: RingHom[S, T],
    s: Set[U]
) {
    ring_hom_preimage(compose_ring_hom(f, g), s) =
        ring_hom_preimage(g, ring_hom_preimage(f, s))
} by {
    compose_ring_hom_hom(f, g)
    ring_hom_preimage(compose_ring_hom(f, g), s) =
        set_preimage(compose(f.hom, g.hom), s)
    ring_hom_preimage(f, s) = set_preimage(f.hom, s)
    ring_hom_preimage(g, ring_hom_preimage(f, s)) =
        set_preimage(g.hom, set_preimage(f.hom, s))
    set_preimage_compose(f.hom, g.hom, s)
}

/// The source closure induced by a ring homomorphism contains the original set.
theorem ring_hom_image_preimage_closure_extensive[S: Ring, T: Ring](
    f: RingHom[S, T],
    s: Set[S]
) {
    s.subset(ring_hom_image_preimage_closure(f, s))
} by {
    ring_hom_image_preimage_closure(f, s) = set_image_preimage_closure(f.hom, s)
    set_image_preimage_closure_extensive(f.hom, s)
}

/// The source closure induced by a ring homomorphism preserves inclusion.
theorem ring_hom_image_preimage_closure_monotone[S: Ring, T: Ring](
    f: RingHom[S, T]
) {
    is_subset_monotone_map(ring_hom_image_preimage_closure(f))
} by {
    forall(s: Set[S], u: Set[S]) {
        if s.subset(u) {
            ring_hom_image_preimage_closure(f, s) = set_image_preimage_closure(f.hom, s)
            ring_hom_image_preimage_closure(f, u) = set_image_preimage_closure(f.hom, u)
            set_image_preimage_closure_monotone(f.hom, s, u)
            ring_hom_image_preimage_closure(f, s).subset(
                ring_hom_image_preimage_closure(f, u)
            )
        }
    }
}

/// The source closure induced by a ring homomorphism is idempotent.
theorem ring_hom_image_preimage_closure_idempotent[S: Ring, T: Ring](
    f: RingHom[S, T],
    s: Set[S]
) {
    ring_hom_image_preimage_closure(f, ring_hom_image_preimage_closure(f, s)) =
        ring_hom_image_preimage_closure(f, s)
} by {
    ring_hom_image_preimage_closure(f, s) = set_image_preimage_closure(f.hom, s)
    ring_hom_image_preimage_closure(f, ring_hom_image_preimage_closure(f, s)) =
        set_image_preimage_closure(f.hom, set_image_preimage_closure(f.hom, s))
    set_image_preimage_closure_idempotent(f.hom, s)
}

/// The source closure induced by a ring homomorphism is a set closure operator.
theorem ring_hom_image_preimage_closure_operator[S: Ring, T: Ring](
    f: RingHom[S, T]
) {
    is_set_closure_operator(ring_hom_image_preimage_closure(f))
} by {
    ring_hom_image_preimage_closure_monotone(f)
    is_subset_monotone_map(ring_hom_image_preimage_closure(f))
    forall(s: Set[S]) {
        ring_hom_image_preimage_closure_extensive(f, s)
        s.subset(ring_hom_image_preimage_closure(f, s))
    }
    is_set_extensive_map(ring_hom_image_preimage_closure(f))
    forall(s: Set[S]) {
        ring_hom_image_preimage_closure_idempotent(f, s)
        ring_hom_image_preimage_closure(f, ring_hom_image_preimage_closure(f, s)) =
            ring_hom_image_preimage_closure(f, s)
    }
    is_set_idempotent_map(ring_hom_image_preimage_closure(f)) = forall(s: Set[S]) {
        ring_hom_image_preimage_closure(f, ring_hom_image_preimage_closure(f, s)) =
            ring_hom_image_preimage_closure(f, s)
    }
    is_set_idempotent_map(ring_hom_image_preimage_closure(f))
    is_set_closure_operator(ring_hom_image_preimage_closure(f)) =
        (is_subset_monotone_map(ring_hom_image_preimage_closure(f)) and
        is_set_extensive_map(ring_hom_image_preimage_closure(f)) and
        is_set_idempotent_map(ring_hom_image_preimage_closure(f)))
    is_set_closure_operator(ring_hom_image_preimage_closure(f))
}

/// The target kernel induced by a ring homomorphism lies in the original set.
theorem ring_hom_preimage_image_kernel_reductive[S: Ring, T: Ring](
    f: RingHom[S, T],
    s: Set[T]
) {
    ring_hom_preimage_image_kernel(f, s).subset(s)
} by {
    ring_hom_preimage_image_kernel(f, s) = set_preimage_image_kernel(f.hom, s)
    set_preimage_image_kernel_reductive(f.hom, s)
}

/// The target kernel induced by a ring homomorphism preserves inclusion.
theorem ring_hom_preimage_image_kernel_monotone[S: Ring, T: Ring](
    f: RingHom[S, T]
) {
    is_subset_monotone_map(ring_hom_preimage_image_kernel(f))
} by {
    forall(s: Set[T], u: Set[T]) {
        if s.subset(u) {
            ring_hom_preimage_image_kernel(f, s) = set_preimage_image_kernel(f.hom, s)
            ring_hom_preimage_image_kernel(f, u) = set_preimage_image_kernel(f.hom, u)
            set_preimage_image_kernel_monotone(f.hom, s, u)
            set_preimage_image_kernel(f.hom, s).subset(set_preimage_image_kernel(f.hom, u))
            ring_hom_preimage_image_kernel(f, s).subset(
                ring_hom_preimage_image_kernel(f, u)
            )
        }
    }
}

/// The target kernel induced by a ring homomorphism is idempotent.
theorem ring_hom_preimage_image_kernel_idempotent[S: Ring, T: Ring](
    f: RingHom[S, T],
    s: Set[T]
) {
    ring_hom_preimage_image_kernel(f, ring_hom_preimage_image_kernel(f, s)) =
        ring_hom_preimage_image_kernel(f, s)
} by {
    ring_hom_preimage_image_kernel(f, s) = set_preimage_image_kernel(f.hom, s)
    ring_hom_preimage_image_kernel(f, ring_hom_preimage_image_kernel(f, s)) =
        set_preimage_image_kernel(f.hom, set_preimage_image_kernel(f.hom, s))
    set_preimage_image_kernel_idempotent(f.hom, s)
}

/// The target kernel induced by a ring homomorphism is a set kernel operator.
theorem ring_hom_preimage_image_kernel_operator[S: Ring, T: Ring](
    f: RingHom[S, T]
) {
    is_set_kernel_operator(ring_hom_preimage_image_kernel(f))
} by {
    ring_hom_preimage_image_kernel_monotone(f)
    is_subset_monotone_map(ring_hom_preimage_image_kernel(f))
    forall(s: Set[T]) {
        ring_hom_preimage_image_kernel_reductive(f, s)
        ring_hom_preimage_image_kernel(f, s).subset(s)
    }
    is_set_reductive_map(ring_hom_preimage_image_kernel(f)) = forall(s: Set[T]) {
        ring_hom_preimage_image_kernel(f, s).subset(s)
    }
    is_set_reductive_map(ring_hom_preimage_image_kernel(f))
    forall(s: Set[T]) {
        ring_hom_preimage_image_kernel_idempotent(f, s)
        ring_hom_preimage_image_kernel(f, ring_hom_preimage_image_kernel(f, s)) =
            ring_hom_preimage_image_kernel(f, s)
    }
    is_set_idempotent_map(ring_hom_preimage_image_kernel(f)) = forall(s: Set[T]) {
        ring_hom_preimage_image_kernel(f, ring_hom_preimage_image_kernel(f, s)) =
            ring_hom_preimage_image_kernel(f, s)
    }
    is_set_idempotent_map(ring_hom_preimage_image_kernel(f))
    is_set_kernel_operator(ring_hom_preimage_image_kernel(f)) =
        (is_subset_monotone_map(ring_hom_preimage_image_kernel(f)) and
        is_set_reductive_map(ring_hom_preimage_image_kernel(f)) and
        is_set_idempotent_map(ring_hom_preimage_image_kernel(f)))
    is_set_kernel_operator(ring_hom_preimage_image_kernel(f))
}
