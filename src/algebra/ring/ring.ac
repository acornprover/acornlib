from nat import Nat, two_divides_suc_iff
from list import sum, map
from algebra.add_comm_group import AddCommGroup
from semiring import Semiring

/// A ring is a structure with two operations (addition and multiplication) where addition forms an abelian group,
/// multiplication forms a monoid, and multiplication distributes over addition.
typeclass Ring extends Semiring, AddCommGroup

theorem mul_zero_left[R: Ring](a: R) {
    R.0 * a = R.0
}

theorem mul_zero_right[R: Ring](a: R) {
    a * R.0 = R.0
}

theorem mul_neg_left[R: Ring](a: R, b: R) {
    -a * b = -(a * b)
} by {
    -a * b + a * b = R.0
}

theorem mul_neg_right[R: Ring](a: R, b: R) {
    a * -b = -(a * b)
} by {
    a * -b + a * b = R.0
}

theorem mul_neg_neg[R: Ring](a: R, b: R) {
    -a * -b = a * b
}

/// Multiplication distributes over subtraction on the left.
theorem mul_sub_left[R: Ring](a: R, b: R, c: R) {
    a * (b - c) = a * b - a * c
} by {
    a * (b - c) = a * (b + -c)
    a * (b + -c) = a * b + a * -c
    mul_neg_right[R](a, c)
    a * -c = -(a * c)
    a * b + -(a * c) = a * b - a * c
}

/// Multiplication distributes over subtraction on the right.
theorem mul_sub_right[R: Ring](a: R, b: R, c: R) {
    (a - b) * c = a * c - b * c
} by {
    (a - b) * c = (a + -b) * c
    (a + -b) * c = a * c + -b * c
    mul_neg_left[R](b, c)
    -b * c = -(b * c)
    a * c + -(b * c) = a * c - b * c
}

/// The product of two negative ones is one.
theorem neg_one_mul_neg_one[R: Ring] {
    -R.1 * -R.1 = R.1
} by {
    mul_neg_neg[R](R.1, R.1)
    -R.1 * -R.1 = R.1 * R.1
    R.1 * R.1 = R.1
}

theorem mul_neg_one_left[R: Ring](a: R) {
    -R.1 * a = -a
}

theorem mul_neg_one_right[R: Ring](a: R) {
    a * -R.1 = -a
}

/// The alternating sign `(-1)^n` in a ring.
define alternating_sign[R: Ring](n: Nat) -> R {
    match n {
        Nat.zero {
            R.1
        }
        Nat.suc(k) {
            -alternating_sign[R](k)
        }
    }
}

/// The zeroth alternating sign is one.
theorem alternating_sign_zero[R: Ring] {
    alternating_sign[R](Nat.0) = R.1
}

/// A successor flips the alternating sign.
theorem alternating_sign_suc[R: Ring](n: Nat) {
    alternating_sign[R](n.suc) = -alternating_sign[R](n)
}

/// Multiplication by `-1` flips the alternating sign.
theorem alternating_sign_suc_mul[R: Ring](n: Nat) {
    alternating_sign[R](n.suc) = -R.1 * alternating_sign[R](n)
} by {
    alternating_sign_suc[R](n)
    mul_neg_one_left[R](alternating_sign[R](n))
}

/// The alternating sign agrees with the monoid power of `-1`.
theorem alternating_sign_eq_neg_one_pow[R: Ring](n: Nat) {
    alternating_sign[R](n) = (-R.1).pow(n)
} by {
    define p(k: Nat) -> Bool {
        alternating_sign[R](k) = (-R.1).pow(k)
    }

    alternating_sign_zero[R]
    (-R.1).pow(Nat.0) = R.1
    p(Nat.0)

    forall(k: Nat) {
        if p(k) {
            alternating_sign_suc[R](k)
            alternating_sign[R](k.suc) = -alternating_sign[R](k)
            alternating_sign[R](k) = (-R.1).pow(k)
            alternating_sign[R](k.suc) = -(-R.1).pow(k)
            mul_neg_one_left[R]((-R.1).pow(k))
            -R.1 * (-R.1).pow(k) = -(-R.1).pow(k)
            (-R.1).pow(k.suc) = -R.1 * (-R.1).pow(k)
            alternating_sign[R](k.suc) = (-R.1).pow(k.suc)
            p(k.suc)
        }
    }
    p(n)
}

/// The alternating sign is one at even indices and negative one at odd indices.
theorem alternating_sign_parity[R: Ring](n: Nat) {
    (Nat.2.divides(n) implies alternating_sign[R](n) = R.1) and
    (not Nat.2.divides(n) implies alternating_sign[R](n) = -R.1)
} by {
    define p(k: Nat) -> Bool {
        (Nat.2.divides(k) implies alternating_sign[R](k) = R.1) and
        (not Nat.2.divides(k) implies alternating_sign[R](k) = -R.1)
    }

    alternating_sign_zero[R]
    p(Nat.0)

    forall(k: Nat) {
        if p(k) {
            two_divides_suc_iff(k)
            alternating_sign_suc[R](k)
            if Nat.2.divides(k.suc) {
                not Nat.2.divides(k)
                alternating_sign[R](k) = -R.1
                alternating_sign[R](k.suc) = -(-R.1)
                alternating_sign[R](k.suc) = R.1
            } else {
                Nat.2.divides(k)
                alternating_sign[R](k) = R.1
                alternating_sign[R](k.suc) = -R.1
            }
            p(k.suc)
        }
    }
    p(Nat.0) and forall(k: Nat) {
        p(k) implies p(k.suc)
    }
    Nat.induction(p)
    p(n)
}

theorem geometric_sum[R: Ring](a: R, n: Nat) {
    (a + -R.1) * sum(map(n.range, a.pow)) = a.pow(n) + -R.1
} by {
    define f(x: Nat) -> Bool {
        (a + -R.1) * sum(map(x.range, a.pow)) = a.pow(x) + -R.1
    }

    // Base case: n = 0
    // range(0) = []
    // sum(map([], a.pow)) = 0
    // (a + -R.1) * 0 = 0
    // pow(a, 0) + -R.1 = R.1 + -R.1 = 0
    f(Nat.0)

    // Inductive step
    forall(x: Nat) {
        if f(x) {
            // Induction hypothesis: (a + -R.1) * sum(map(x.range, a.pow)) = a.pow(x) + -R.1

            // range(x.suc) = range(x).append(x)

            // sum of appended list

            // Left side: (a + -R.1) * sum(map(range(x.suc), a.pow))

            // Use induction hypothesis

            (a + -R.1) * (lib(list).partial[R](a.pow, x) + a.pow(x)) = (a + -R.1) * lib(list).partial[R](a.pow, x) + (a + -R.1) * a.pow(x)

            // Simplify (a + -R.1) * a.pow(x)
            (a + -R.1) * a.pow(x) = a * a.pow(x) + -R.1 * a.pow(x)
            a * a.pow(x) = a.pow(x.suc)
            -R.1 * a.pow(x) = -a.pow(x)
            (a + -R.1) * a.pow(x) = a.pow(x.suc) + -a.pow(x)

            // Combine terms
            (a + -R.1) * lib(list).partial[R](a.pow, x.suc) = (a + -R.1) * (lib(list).partial[R](a.pow, x) + a.pow(x))
            (a + -R.1) * lib(list).partial[R](a.pow, x) + (a + -R.1) * a.pow(x) = (a.pow(x) + -R.1) + (a.pow(x.suc) + -a.pow(x))
            (a.pow(x) + -R.1) + (a.pow(x.suc) + -a.pow(x)) = a.pow(x) + -R.1 + a.pow(x.suc) + -a.pow(x)
            a.pow(x) + -R.1 + a.pow(x.suc) + -a.pow(x) = a.pow(x.suc) + (a.pow(x) + -a.pow(x)) + -R.1
            a.pow(x.suc) + (a.pow(x) + -a.pow(x)) + -R.1 = a.pow(x.suc) + R.0 + -R.1
            a.pow(x.suc) + R.0 = a.pow(x.suc)
            a.pow(x.suc) + R.0 + -R.1 = a.pow(x.suc) + -R.1
            -a.pow(x) + (a.pow(x) + -R.1 + a.pow(x.suc)) = -a.pow(x) + (a.pow(x) + -R.1) + a.pow(x.suc)
            -a.pow(x) + (a.pow(x) + -R.1) = -a.pow(x) + a.pow(x) + -R.1
            -a.pow(x) + (a.pow(x) + -R.1 + a.pow(x.suc)) = a.pow(x) + -R.1 + a.pow(x.suc) + -a.pow(x)
            -a.pow(x) + a.pow(x) = R.0
            R.0 + -R.1 = -R.1
            a.pow(x) + -R.1 + a.pow(x.suc) + -a.pow(x) = -R.1 + a.pow(x.suc)

            // Therefore
            (a + -R.1) * sum(map(x.range, a.pow)) = a.pow(x) + -R.1
            sum(map(x.range, a.pow)) = lib(list).partial[R](a.pow, x)
            sum(map(x.suc.range, a.pow)) = lib(list).partial[R](a.pow, x.suc)
            lib(list).partial[R](a.pow, x) + a.pow(x) = lib(list).partial[R](a.pow, x.suc)
            (a + -R.1) * lib(list).partial[R](a.pow, x) + (a + -R.1) * a.pow(x) = (a + -R.1) * (lib(list).partial[R](a.pow, x) + a.pow(x))
            -R.1 + a.pow(x.suc) = a.pow(x.suc) + -R.1
            (a + -R.1) * sum(map(x.suc.range, a.pow)) = a.pow(x.suc) + -R.1
        }
    }

    f(n)
}

/// Geometric-sum endpoint using subtraction notation.
theorem geometric_sum_sub_one[R: Ring](a: R, n: Nat) {
    (a - R.1) * sum(map(n.range, a.pow)) = a.pow(n) - R.1
} by {
    a - R.1 = a + -R.1
    a.pow(n) - R.1 = a.pow(n) + -R.1
    geometric_sum[R](a, n)
}
