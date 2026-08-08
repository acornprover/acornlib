from nat import Nat, pow_zero
from algebra.ring.ring import Ring
from algebra.add_monoid import is_add_monoid_hom
from algebra.add_group import AddGroup, AddGroupHom, is_add_group_hom, left_cancel, add_group_hom_add
from algebra.monoid.monoid import Monoid, MonoidHom, is_monoid_hom, monoid_hom_one, monoid_hom_mul
from algebra.semiring_hom import SemiringHom, is_semiring_hom
from data.basic.functions import identity_fn, compose, compose_assoc, compose_identity_left, compose_identity_right,
    function_eq_transport_predicate, function_eq_transport_predicate_rev
from data.basic.relation_transport import preserves_binary_op

/// True if a function between rings preserves both ring operations and the multiplicative identity.
define is_ring_hom[R: Ring, S: Ring](f: R -> S) -> Bool {
    f(R.1) = S.1 and forall(a: R, b: R) {
        f(a + b) = f(a) + f(b) and f(a * b) = f(a) * f(b)
    }
}

/// A ring homomorphism that preserves both addition and multiplication and the multiplicative identity.
structure RingHom[R: Ring, S: Ring] {
    /// The mapping for the homomorphism.
    hom: R -> S
} constraint {
    is_ring_hom(hom)
}

/// Ring homomorphism extensionality: two homomorphisms are equal when they agree on every input.
theorem ring_hom_ext[R: Ring, S: Ring](f: RingHom[R, S], g: RingHom[R, S]) {
    (forall(a: R) { f.hom(a) = g.hom(a) }) implies f = g
} by {
    if forall(a: R) { f.hom(a) = g.hom(a) } {
        f.hom = g.hom
    }
}

attributes RingHom[R: Ring, S: Ring] {
    /// Ring homomorphism extensionality from pointwise equality of the homomorphism.
    let ext = ring_hom_ext[R, S]
}

/// Equal ring homomorphisms have equal underlying functions.
theorem ring_hom_eq_hom[R: Ring, S: Ring](f: RingHom[R, S], g: RingHom[R, S]) {
    f = g implies f.hom = g.hom
}

/// Equal ring homomorphisms have equal values at every element.
theorem ring_hom_eq_apply[R: Ring, S: Ring](f: RingHom[R, S], g: RingHom[R, S], a: R) {
    f = g implies f.hom(a) = g.hom(a)
} by {
    if f = g {
        f.hom = g.hom
        f.hom(a) = g.hom(a)
    }
}

/// Equality of ring homomorphisms transports predicates on ring homomorphisms.
theorem ring_hom_eq_transport_predicate[R: Ring, S: Ring](
    p: RingHom[R, S] -> Bool,
    f: RingHom[R, S],
    g: RingHom[R, S]
) {
    f = g and p(f) implies p(g)
}

/// Equality of ring homomorphisms transports predicates on ring homomorphisms in the reverse direction.
theorem ring_hom_eq_transport_predicate_rev[R: Ring, S: Ring](
    p: RingHom[R, S] -> Bool,
    f: RingHom[R, S],
    g: RingHom[R, S]
) {
    f = g and p(g) implies p(f)
}

/// Equality of underlying functions determines equality of ring homomorphisms.
theorem ring_hom_eq_of_hom_eq[R: Ring, S: Ring](f: RingHom[R, S], g: RingHom[R, S]) {
    f.hom = g.hom implies f = g
} by {
    if f.hom = g.hom {
        forall(a: R) {
            f.hom(a) = g.hom(a)
        }
        ring_hom_ext(f, g)
    }
}

/// Pointwise equality determines equality of ring homomorphisms.
theorem ring_hom_eq_of_apply_eq[R: Ring, S: Ring](f: RingHom[R, S], g: RingHom[R, S]) {
    (forall(a: R) { f.hom(a) = g.hom(a) }) implies f = g
} by {
    if forall(a: R) { f.hom(a) = g.hom(a) } {
        ring_hom_ext(f, g)
    }
}

/// A ring homomorphism preserves the multiplicative identity.
theorem ring_hom_one[R: Ring, S: Ring](f: RingHom[R, S]) {
    f.hom(R.1) = S.1
}

/// A ring homomorphism preserves addition.
theorem ring_hom_add[R: Ring, S: Ring](f: RingHom[R, S], a: R, b: R) {
    f.hom(a + b) = f.hom(a) + f.hom(b)
} by {
    is_ring_hom(f.hom)
    f.hom(R.1) = S.1 and forall(x: R, y: R) {
        f.hom(x + y) = f.hom(x) + f.hom(y) and f.hom(x * y) = f.hom(x) * f.hom(y)
    }
}

/// A ring homomorphism preserves multiplication.
theorem ring_hom_mul[R: Ring, S: Ring](f: RingHom[R, S], a: R, b: R) {
    f.hom(a * b) = f.hom(a) * f.hom(b)
} by {
    is_ring_hom(f.hom)
    f.hom(R.1) = S.1 and forall(x: R, y: R) {
        f.hom(x + y) = f.hom(x) + f.hom(y) and f.hom(x * y) = f.hom(x) * f.hom(y)
    }
}

/// A ring homomorphism preserves the additive identity.
theorem ring_hom_zero[R: Ring, S: Ring](f: RingHom[R, S]) {
    f.hom(R.0) = S.0
} by {
    R.0 + R.0 = R.0
    f.hom(R.0 + R.0) = f.hom(R.0)
    ring_hom_add(f, R.0, R.0)
    f.hom(R.0) + f.hom(R.0) = f.hom(R.0)
    f.hom(R.0) + f.hom(R.0) = f.hom(R.0) + S.0
    left_cancel(f.hom(R.0), f.hom(R.0), S.0)
}

/// A ring homomorphism preserves additive inverses.
theorem ring_hom_neg[R: Ring, S: Ring](f: RingHom[R, S], a: R) {
    f.hom(-a) = -f.hom(a)
} by {
    a + -a = R.0
    f.hom(a + -a) = f.hom(R.0)
    ring_hom_add(f, a, -a)
    f.hom(a) + f.hom(-a) = f.hom(R.0)
    ring_hom_zero(f)
    f.hom(a) + f.hom(-a) = S.0
    f.hom(a) + -f.hom(a) = S.0
    f.hom(a) + f.hom(-a) = f.hom(a) + -f.hom(a)
    left_cancel(f.hom(a), f.hom(-a), -f.hom(a))
}

/// A ring homomorphism is an additive group homomorphism.
theorem ring_hom_is_add_group_hom[R: Ring, S: Ring](f: RingHom[R, S]) {
    is_add_group_hom(f.hom)
} by {
    is_ring_hom(f.hom)
    forall(a: R, b: R) {
        ring_hom_add(f, a, b)
        f.hom(a + b) = f.hom(a) + f.hom(b)
    }
}

/// A ring homomorphism is a multiplicative monoid homomorphism.
theorem ring_hom_is_monoid_hom[R: Ring, S: Ring](f: RingHom[R, S]) {
    is_monoid_hom(f.hom)
} by {
    is_ring_hom(f.hom)
    ring_hom_one(f)
    f.hom(R.1) = S.1
    forall(a: R, b: R) {
        ring_hom_mul(f, a, b)
        f.hom(a * b) = f.hom(a) * f.hom(b)
    }
    is_monoid_hom(f.hom)
}

/// A ring homomorphism is a semiring homomorphism.
theorem ring_hom_is_semiring_hom[R: Ring, S: Ring](f: RingHom[R, S]) {
    is_semiring_hom(f.hom)
} by {
    ring_hom_is_add_group_hom(f)
    is_add_group_hom(f.hom)
    ring_hom_is_monoid_hom(f)
    is_monoid_hom(f.hom)
    ring_hom_zero(f)
    f.hom(R.0) = S.0
    forall(a: R, b: R) {
        ring_hom_add(f, a, b)
        f.hom(a + b) = f.hom(a) + f.hom(b)
    }
    is_add_monoid_hom(f.hom)
    is_add_monoid_hom(f.hom) and is_monoid_hom(f.hom)
    is_semiring_hom(f.hom)
}

/// The additive group homomorphism underlying a ring homomorphism.
let ring_hom_to_add_group_hom[R: Ring, S: Ring](f: RingHom[R, S]) -> result: AddGroupHom[R, S] satisfy {
    AddGroupHom.new(f.hom) = Option.some(result)
} by {
    ring_hom_is_add_group_hom(f)
}

/// The underlying function of the additive group homomorphism associated to a ring homomorphism.
theorem ring_hom_to_add_group_hom_hom[R: Ring, S: Ring](f: RingHom[R, S]) {
    ring_hom_to_add_group_hom(f).hom = f.hom
}

/// The multiplicative monoid homomorphism underlying a ring homomorphism.
let ring_hom_to_monoid_hom[R: Ring, S: Ring](f: RingHom[R, S]) -> result: MonoidHom[R, S] satisfy {
    MonoidHom.new(f.hom) = Option.some(result)
} by {
    ring_hom_is_monoid_hom(f)
}

/// The underlying function of the monoid homomorphism associated to a ring homomorphism.
theorem ring_hom_to_monoid_hom_hom[R: Ring, S: Ring](f: RingHom[R, S]) {
    ring_hom_to_monoid_hom(f).hom = f.hom
}

/// The semiring homomorphism underlying a ring homomorphism.
let ring_hom_to_semiring_hom[R: Ring, S: Ring](f: RingHom[R, S]) -> result: SemiringHom[R, S] satisfy {
    SemiringHom.new(f.hom) = Option.some(result)
} by {
    ring_hom_is_semiring_hom(f)
}

/// The underlying function of the semiring homomorphism associated to a ring homomorphism.
theorem ring_hom_to_semiring_hom_hom[R: Ring, S: Ring](f: RingHom[R, S]) {
    ring_hom_to_semiring_hom(f).hom = f.hom
}

/// Compatible additive and multiplicative homomorphisms determine a ring homomorphism.
theorem add_group_hom_monoid_hom_is_ring_hom[R: Ring, S: Ring](f: AddGroupHom[R, S], g: MonoidHom[R, S]) {
    f.hom = g.hom implies is_ring_hom(f.hom)
} by {
    if f.hom = g.hom {
        monoid_hom_one(g)
        g.hom(R.1) = S.1
        f.hom(R.1) = S.1
        forall(a: R, b: R) {
            add_group_hom_add(f, a, b)
            f.hom(a + b) = f.hom(a) + f.hom(b)
            monoid_hom_mul(g, a, b)
            g.hom(a * b) = g.hom(a) * g.hom(b)
            f.hom(a * b) = f.hom(a) * f.hom(b)
            f.hom(a + b) = f.hom(a) + f.hom(b) and f.hom(a * b) = f.hom(a) * f.hom(b)
        }
        is_ring_hom(f.hom)
    }
}

/// A ring homomorphism preserves powers.
theorem ring_hom_pow[R: Ring, S: Ring](f: RingHom[R, S], a: R, n: Nat) {
    f.hom(a.pow(n)) = f.hom(a).pow(n)
} by {
    define p(k: Nat) -> Bool {
        f.hom(a.pow(k)) = f.hom(a).pow(k)
    }

    a.pow(Nat.0) = R.1
    f.hom(R.1) = S.1
    f.hom(a).pow(Nat.0) = S.1
    p(Nat.0)

    forall(k: Nat) {
        if p(k) {
            f.hom(a.pow(k)) = f.hom(a).pow(k)
            a.pow(k.suc) = a * a.pow(k)
            ring_hom_mul(f, a, a.pow(k))
            f.hom(a * a.pow(k)) = f.hom(a) * f.hom(a.pow(k))
            f.hom(a) * f.hom(a.pow(k)) = f.hom(a) * f.hom(a).pow(k)
            f.hom(a) * f.hom(a).pow(k) = f.hom(a).pow(k.suc)
            p(k.suc)
        }
    }
}

/// Equality of functions transports the property of being a ring homomorphism.
theorem function_eq_transport_ring_hom[R: Ring, S: Ring](f: R -> S, g: R -> S) {
    f = g and is_ring_hom(f) implies is_ring_hom(g)
} by {
    if f = g and is_ring_hom(f) {
        function_eq_transport_predicate(is_ring_hom[R, S], f, g)
    }
}

/// Equality of functions transports the property of being a ring homomorphism in the reverse direction.
theorem function_eq_transport_ring_hom_rev[R: Ring, S: Ring](f: R -> S, g: R -> S) {
    f = g and is_ring_hom(g) implies is_ring_hom(f)
} by {
    if f = g and is_ring_hom(g) {
        function_eq_transport_predicate_rev(is_ring_hom[R, S], f, g)
    }
}

/// The identity function on a ring preserves the ring structure.
theorem identity_fn_is_ring_hom[R: Ring] {
    is_ring_hom(identity_fn[R])
} by {
    identity_fn[R](R.1) = R.1
    forall(a: R, b: R) {
        identity_fn[R](a + b) = a + b
        identity_fn[R](a) = a
        identity_fn[R](b) = b
        identity_fn[R](a) + identity_fn[R](b) = a + b
        identity_fn[R](a + b) = identity_fn[R](a) + identity_fn[R](b)
        identity_fn[R](a * b) = a * b
        identity_fn[R](a) * identity_fn[R](b) = a * b
        identity_fn[R](a * b) = identity_fn[R](a) * identity_fn[R](b)
    }
    identity_fn[R](R.1) = R.1 and forall(a: R, b: R) {
        identity_fn[R](a + b) = identity_fn[R](a) + identity_fn[R](b) and
            identity_fn[R](a * b) = identity_fn[R](a) * identity_fn[R](b)
    }
}

/// The composition of two ring homomorphisms preserves the ring structure.
theorem compose_is_ring_hom[R: Ring, S: Ring, T: Ring](f: S -> T, g: R -> S) {
    is_ring_hom(f) and is_ring_hom(g) implies is_ring_hom(compose(f, g))
} by {
    if is_ring_hom(f) and is_ring_hom(g) {
        is_ring_hom(g) = (g(R.1) = S.1 and forall(x: R, y: R) {
            g(x + y) = g(x) + g(y) and g(x * y) = g(x) * g(y)
        })
        is_ring_hom(f) = (f(S.1) = T.1 and forall(x: S, y: S) {
            f(x + y) = f(x) + f(y) and f(x * y) = f(x) * f(y)
        })
        g(R.1) = S.1
        f(S.1) = T.1
        compose(f, g)(R.1) = f(g(R.1))
        compose(f, g)(R.1) = T.1
        forall(a: R, b: R) {
            compose(f, g)(a + b) = f(g(a + b))
            g(a + b) = g(a) + g(b)
            f(g(a) + g(b)) = f(g(a)) + f(g(b))
            compose(f, g)(a) = f(g(a))
            compose(f, g)(b) = f(g(b))
            compose(f, g)(a + b) = compose(f, g)(a) + compose(f, g)(b)
            compose(f, g)(a * b) = f(g(a * b))
            g(a * b) = g(a) * g(b)
            f(g(a) * g(b)) = f(g(a)) * f(g(b))
            compose(f, g)(a * b) = compose(f, g)(a) * compose(f, g)(b)
            compose(f, g)(a + b) = compose(f, g)(a) + compose(f, g)(b) and
                compose(f, g)(a * b) = compose(f, g)(a) * compose(f, g)(b)
        }
        compose(f, g)(R.1) = T.1 and forall(a: R, b: R) {
            compose(f, g)(a + b) = compose(f, g)(a) + compose(f, g)(b) and
                compose(f, g)(a * b) = compose(f, g)(a) * compose(f, g)(b)
        }
        is_ring_hom(compose(f, g))
    }
}

/// The identity homomorphism on a ring, which sends every element to itself.
let identity_ring_hom[R: Ring]: RingHom[R, R] satisfy {
    RingHom.new(identity_fn[R]) = Option.some(identity_ring_hom)
}

/// The underlying function of the identity ring homomorphism is the identity function.
theorem identity_ring_hom_hom[R: Ring] {
    identity_ring_hom[R].hom = identity_fn[R]
}

/// The composition of two ring homomorphisms.
let compose_ring_hom[R: Ring, S: Ring, T: Ring](f: RingHom[S, T], g: RingHom[R, S]) -> result: RingHom[R, T] satisfy {
    RingHom.new(compose(f.hom, g.hom)) = Option.some(result)
} by {
    is_ring_hom(f.hom)
    is_ring_hom(g.hom)
    compose_is_ring_hom(f.hom, g.hom)
    is_ring_hom(compose(f.hom, g.hom))
}

/// The underlying function of a composition of ring homomorphisms is the composition of underlying functions.
theorem compose_ring_hom_hom[R: Ring, S: Ring, T: Ring](f: RingHom[S, T], g: RingHom[R, S]) {
    compose_ring_hom(f, g).hom = compose(f.hom, g.hom)
}

/// Composition of ring homomorphisms is associative.
theorem compose_ring_hom_assoc[R: Ring, S: Ring, T: Ring, U: Ring](
    f: RingHom[T, U], g: RingHom[S, T], h: RingHom[R, S]) {
    compose_ring_hom(compose_ring_hom(f, g), h) = compose_ring_hom(f, compose_ring_hom(g, h))
} by {
    let lhs = compose_ring_hom(compose_ring_hom(f, g), h)
    let rhs = compose_ring_hom(f, compose_ring_hom(g, h))
    lhs.hom = compose(compose_ring_hom(f, g).hom, h.hom)
    compose_ring_hom(f, g).hom = compose(f.hom, g.hom)
    lhs.hom = compose(compose(f.hom, g.hom), h.hom)
    rhs.hom = compose(f.hom, compose_ring_hom(g, h).hom)
    compose_ring_hom(g, h).hom = compose(g.hom, h.hom)
    rhs.hom = compose(f.hom, compose(g.hom, h.hom))
    compose_assoc(f.hom, g.hom, h.hom)
    lhs.hom = rhs.hom
}

/// Composing the identity ring homomorphism on the left does nothing.
theorem compose_ring_hom_identity_left[R: Ring, S: Ring](f: RingHom[R, S]) {
    compose_ring_hom(identity_ring_hom[S], f) = f
} by {
    let lhs = compose_ring_hom(identity_ring_hom[S], f)
    lhs.hom = compose(identity_ring_hom[S].hom, f.hom)
    identity_ring_hom[S].hom = identity_fn[S]
    lhs.hom = compose(identity_fn[S], f.hom)
    compose_identity_left(f.hom)
    lhs.hom = f.hom
}

/// Composing the identity ring homomorphism on the right does nothing.
theorem compose_ring_hom_identity_right[R: Ring, S: Ring](f: RingHom[R, S]) {
    compose_ring_hom(f, identity_ring_hom[R]) = f
} by {
    let lhs = compose_ring_hom(f, identity_ring_hom[R])
    lhs.hom = compose(f.hom, identity_ring_hom[R].hom)
    identity_ring_hom[R].hom = identity_fn[R]
    lhs.hom = compose(f.hom, identity_fn[R])
    compose_identity_right(f.hom)
    lhs.hom = f.hom
}
