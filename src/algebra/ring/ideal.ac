from comm_ring import CommRing
from algebra.ring.ring_hom import RingHom, ring_hom_zero, ring_hom_one, ring_hom_add, ring_hom_mul
from data.basic.functions import predicate_extensionality
from data.basic.set import Set, singleton_set_is_finite, is_subset_monotone_map, is_set_extensive_map,
    is_set_idempotent_map, is_set_closure_operator, indexed_intersection,
    indexed_intersection_contains_of_forall, indexed_intersection_contains_at, set_ext

/// True if a subset of a commutative ring contains the additive identity.
define ideal_zero_constraint[R: CommRing](contains: R -> Bool) -> Bool {
    contains(R.0)
}

/// True if a subset of a commutative ring is closed under addition.
define ideal_add_constraint[R: CommRing](contains: R -> Bool) -> Bool {
    forall(a: R, b: R) {
        contains(a) and contains(b) implies contains(a + b)
    }
}

/// True if a subset of a commutative ring absorbs multiplication by any ring element.
define ideal_absorb_constraint[R: CommRing](contains: R -> Bool) -> Bool {
    forall(r: R, a: R) {
        contains(a) implies contains(r * a)
    }
}

/// True if a subset of a commutative ring is an ideal: it contains zero,
/// is closed under addition, and absorbs multiplication by any ring element.
define is_ideal[R: CommRing](contains: R -> Bool) -> Bool {
    ideal_zero_constraint(contains)
    and ideal_add_constraint(contains)
    and ideal_absorb_constraint(contains)
}

/// is_ideal unfolds to the conjunction of its three sub-constraints.
theorem is_ideal_unfold[R: CommRing](contains: R -> Bool) {
    is_ideal(contains) = (ideal_zero_constraint(contains) and ideal_add_constraint(contains) and ideal_absorb_constraint(contains))
}

/// The three ideal sub-constraints together establish is_ideal.
theorem is_ideal_from_constraints[R: CommRing](contains: R -> Bool) {
    ideal_zero_constraint(contains) and ideal_add_constraint(contains) and ideal_absorb_constraint(contains)
        implies is_ideal(contains)
} by {
    is_ideal_unfold(contains)
}

/// An ideal contains the additive identity.
theorem is_ideal_zero[R: CommRing](contains: R -> Bool) {
    is_ideal(contains) implies contains(R.0)
} by {
    if is_ideal(contains) {
        is_ideal(contains) = (ideal_zero_constraint(contains) and ideal_add_constraint(contains) and ideal_absorb_constraint(contains))
        contains(R.0)
    }
}

/// An ideal is closed under addition.
theorem is_ideal_add[R: CommRing](contains: R -> Bool, a: R, b: R) {
    is_ideal(contains) and contains(a) and contains(b) implies contains(a + b)
} by {
    if is_ideal(contains) and contains(a) and contains(b) {
        is_ideal(contains) = (ideal_zero_constraint(contains) and ideal_add_constraint(contains) and ideal_absorb_constraint(contains))
        ideal_add_constraint(contains) = forall(x: R, y: R) {
            contains(x) and contains(y) implies contains(x + y)
        }
        contains(a + b)
    }
}

/// An ideal absorbs multiplication by any ring element.
theorem is_ideal_absorb[R: CommRing](contains: R -> Bool, r: R, a: R) {
    is_ideal(contains) and contains(a) implies contains(r * a)
} by {
    if is_ideal(contains) and contains(a) {
        ideal_absorb_constraint(contains) = forall(s: R, x: R) {
            contains(x) implies contains(s * x)
        }
    }
}

/// An ideal is closed under negation.
theorem is_ideal_neg[R: CommRing](contains: R -> Bool, a: R) {
    is_ideal(contains) and contains(a) implies contains(-a)
} by {
    if is_ideal(contains) and contains(a) {
        is_ideal_absorb(contains, -R.1, a)
        contains(-a)
    }
}

/// The zero subset, containing only the additive identity.
define zero_ideal[R: CommRing](a: R) -> Bool {
    a = R.0
}

/// The zero subset is an ideal.
theorem zero_ideal_is_ideal[R: CommRing] {
    is_ideal(zero_ideal[R])
} by {
    forall(a: R, b: R) {
        if zero_ideal[R](a) and zero_ideal[R](b) {
            a = R.0
            b = R.0
            zero_ideal[R](a + b)
        }
    }
    ideal_add_constraint(zero_ideal[R])
    forall(r: R, a: R) {
        if zero_ideal[R](a) {
            a = R.0
            zero_ideal[R](r * a)
        }
    }
    ideal_absorb_constraint(zero_ideal[R])
}

/// The full ring, the largest ideal.
define unit_ideal[R: CommRing](a: R) -> Bool {
    true
}

/// The full ring is an ideal.
theorem unit_ideal_is_ideal[R: CommRing] {
    is_ideal(unit_ideal[R])
} by {
    forall(a: R, b: R) {
        if unit_ideal[R](a) and unit_ideal[R](b) {
            unit_ideal[R](a + b)
        }
    }
    ideal_add_constraint(unit_ideal[R])
    forall(r: R, a: R) {
        if unit_ideal[R](a) {
            unit_ideal[R](r * a)
        }
    }
    ideal_absorb_constraint(unit_ideal[R])
}

/// An ideal that contains the multiplicative identity is the whole ring.
theorem ideal_eq_unit_of_contains_one[R: CommRing](contains: R -> Bool) {
    is_ideal(contains) and contains(R.1) implies contains = unit_ideal[R]
} by {
    if is_ideal(contains) and contains(R.1) {
        forall(x: R) {
            is_ideal_absorb(contains, x, R.1)
            contains(x) = unit_ideal[R](x)
        }
        predicate_extensionality(contains, unit_ideal[R])
    }
}

/// An ideal contains the multiplicative identity exactly when it is the whole ring.
theorem ideal_contains_one_iff_eq_unit[R: CommRing](contains: R -> Bool) {
    is_ideal(contains) implies (contains(R.1) = (contains = unit_ideal[R]))
} by {
    if is_ideal(contains) {
        if contains(R.1) {
            ideal_eq_unit_of_contains_one(contains)
            contains = unit_ideal[R]
        }
        if contains = unit_ideal[R] {
            unit_ideal[R](R.1)
            contains(R.1)
        }
        contains(R.1) = (contains = unit_ideal[R])
    }
}

/// Membership in the principal ideal generated by an element.
define principal_ideal[R: CommRing](a: R, x: R) -> Bool {
    exists(r: R) {
        x = r * a
    }
}

/// The principal ideal generated by any element is an ideal.
theorem principal_ideal_is_ideal[R: CommRing](a: R) {
    is_ideal(principal_ideal[R](a))
} by {
    principal_ideal(a, R.0)

    forall(x: R, y: R) {
        if principal_ideal(a, x) and principal_ideal(a, y) {
            let r: R satisfy { x = r * a }
            let s: R satisfy { y = s * a }
            x + y = (r + s) * a
            principal_ideal(a, x + y)
        }
    }
    ideal_add_constraint(principal_ideal[R](a))

    forall(r: R, x: R) {
        if principal_ideal(a, x) {
            let s: R satisfy { x = s * a }
            r * x = (r * s) * a
            principal_ideal(a, r * x)
        }
    }
    ideal_absorb_constraint(principal_ideal[R](a))
}

/// The element generating a principal ideal is a member of it.
theorem principal_ideal_contains_generator[R: CommRing](a: R) {
    principal_ideal(a, a)
} by {
}

/// Every multiple of the generator belongs to the principal ideal.
theorem principal_ideal_contains_multiple[R: CommRing](a: R, r: R) {
    principal_ideal(a, r * a)
} by {
    r * a = r * a
}

/// Membership in the principal ideal yields a witnessing multiple.
theorem principal_ideal_witness[R: CommRing](a: R, x: R) {
    principal_ideal(a, x) implies exists(r: R) { x = r * a }
} by {
    if principal_ideal(a, x) {
        exists(r: R) { x = r * a }
    }
}

/// The principal ideal generated by zero is the zero ideal.
theorem principal_ideal_zero_eq_zero_ideal[R: CommRing] {
    principal_ideal[R](R.0) = zero_ideal[R]
} by {
    forall(x: R) {
        if principal_ideal(R.0, x) {
            let r: R satisfy { x = r * R.0 }
            zero_ideal[R](x)
        }
        if zero_ideal[R](x) {
            x = R.0 * R.0
            principal_ideal(R.0, x)
        }
        principal_ideal[R](R.0)(x) = zero_ideal[R](x)
    }
}

/// The principal ideal generated by one is the unit ideal.
theorem principal_ideal_one_eq_unit_ideal[R: CommRing] {
    principal_ideal[R](R.1) = unit_ideal[R]
} by {
    forall(x: R) {
        principal_ideal[R](R.1)(x) = unit_ideal[R](x)
    }
}

/// True if every element of the first subset belongs to the second.
define ideal_subset[R: CommRing](contains_i: R -> Bool, contains_j: R -> Bool) -> Bool {
    forall(x: R) {
        contains_i(x) implies contains_j(x)
    }
}

/// Subset is reflexive.
theorem ideal_subset_refl[R: CommRing](contains: R -> Bool) {
    ideal_subset(contains, contains)
} by {
    forall(x: R) {
        contains(x) implies contains(x)
    }
}

/// Subset is transitive.
theorem ideal_subset_trans[R: CommRing](contains_i: R -> Bool,
                                        contains_j: R -> Bool,
                                        contains_k: R -> Bool) {
    ideal_subset(contains_i, contains_j) and ideal_subset(contains_j, contains_k)
        implies ideal_subset(contains_i, contains_k)
} by {
    if ideal_subset(contains_i, contains_j) and ideal_subset(contains_j, contains_k) {
        forall(x: R) {
            if contains_i(x) {
                contains_j(x)
                contains_k(x)
            }
        }
    }
}

/// Mutual subset implies pointwise equality of the membership predicates.
theorem ideal_subset_antisymm[R: CommRing](contains_i: R -> Bool,
                                           contains_j: R -> Bool) {
    ideal_subset(contains_i, contains_j) and ideal_subset(contains_j, contains_i)
        implies contains_i = contains_j
} by {
    if ideal_subset(contains_i, contains_j) and ideal_subset(contains_j, contains_i) {
        ideal_subset(contains_i, contains_j) = forall(y: R) {
            contains_i(y) implies contains_j(y)
        }
        ideal_subset(contains_j, contains_i) = forall(y: R) {
            contains_j(y) implies contains_i(y)
        }
        forall(x: R) {
            if contains_i(x) {
                contains_j(x)
            }
            if contains_j(x) {
                contains_i(x)
            }
            contains_i(x) = contains_j(x)
        }
        predicate_extensionality(contains_i, contains_j)
        contains_i = contains_j
    }
}

/// Equality of ideal predicates is equivalent to mutual inclusion.
theorem ideal_eq_iff_subset_both[R: CommRing](contains_i: R -> Bool,
                                              contains_j: R -> Bool) {
    contains_i = contains_j = (ideal_subset(contains_i, contains_j) and ideal_subset(contains_j, contains_i))
} by {
    if contains_i = contains_j {
        ideal_subset_refl(contains_i)
        ideal_subset(contains_j, contains_i)
        ideal_subset(contains_i, contains_j) and ideal_subset(contains_j, contains_i)
    }
    if ideal_subset(contains_i, contains_j) and ideal_subset(contains_j, contains_i) {
        ideal_subset_antisymm(contains_i, contains_j)
        contains_i = contains_j
    }
}

/// Any ideal containing the generator contains the whole principal ideal: minimality of `principal_ideal(a)`.
theorem principal_ideal_subset_of_contains_generator[R: CommRing](a: R, contains: R -> Bool) {
    is_ideal(contains) and contains(a) implies
        ideal_subset(principal_ideal[R](a), contains)
} by {
    if is_ideal(contains) and contains(a) {
        forall(x: R) {
            if principal_ideal(a, x) {
                let r: R satisfy { x = r * a }
                is_ideal_absorb(contains, r, a)
                contains(x)
            }
        }
    }
}

/// The zero ideal is contained in every ideal.
theorem zero_ideal_subset[R: CommRing](contains: R -> Bool) {
    is_ideal(contains) implies ideal_subset(zero_ideal[R], contains)
} by {
    if is_ideal(contains) {
        is_ideal_zero(contains)
        forall(x: R) {
            if zero_ideal[R](x) {
                x = R.0
                contains(x)
            }
        }
    }
}

/// Every subset is contained in the unit ideal.
theorem subset_unit_ideal[R: CommRing](contains: R -> Bool) {
    ideal_subset(contains, unit_ideal[R])
} by {
    forall(x: R) {
        unit_ideal[R](x)
    }
}

/// Membership in the intersection of two subsets.
define ideal_inter[R: CommRing](contains_i: R -> Bool,
                                contains_j: R -> Bool,
                                x: R) -> Bool {
    contains_i(x) and contains_j(x)
}

/// The intersection of two ideals satisfies ideal_zero_constraint.
theorem ideal_inter_zero_constraint[R: CommRing](contains_i: R -> Bool,
                                                 contains_j: R -> Bool) {
    is_ideal(contains_i) and is_ideal(contains_j)
        implies ideal_zero_constraint(ideal_inter[R](contains_i, contains_j))
} by {
    if is_ideal(contains_i) and is_ideal(contains_j) {
        is_ideal_zero(contains_i)
        is_ideal_zero(contains_j)
        ideal_inter(contains_i, contains_j, R.0)
    }
}

/// The intersection of two ideals satisfies ideal_add_constraint.
theorem ideal_inter_add_constraint[R: CommRing](contains_i: R -> Bool,
                                                contains_j: R -> Bool) {
    is_ideal(contains_i) and is_ideal(contains_j)
        implies ideal_add_constraint(ideal_inter[R](contains_i, contains_j))
} by {
    if is_ideal(contains_i) and is_ideal(contains_j) {
        forall(a: R, b: R) {
            if ideal_inter(contains_i, contains_j, a) and ideal_inter(contains_i, contains_j, b) {
                contains_i(a) and contains_j(a)
                contains_i(b) and contains_j(b)
                is_ideal_add(contains_i, a, b)
                is_ideal_add(contains_j, a, b)
                ideal_inter(contains_i, contains_j, a + b)
            }
        }
    }
}

/// The intersection of two ideals satisfies ideal_absorb_constraint.
theorem ideal_inter_absorb_constraint[R: CommRing](contains_i: R -> Bool,
                                                   contains_j: R -> Bool) {
    is_ideal(contains_i) and is_ideal(contains_j)
        implies ideal_absorb_constraint(ideal_inter[R](contains_i, contains_j))
} by {
    if is_ideal(contains_i) and is_ideal(contains_j) {
        forall(r: R, a: R) {
            if ideal_inter(contains_i, contains_j, a) {
                contains_i(a) and contains_j(a)
                is_ideal_absorb(contains_i, r, a)
                is_ideal_absorb(contains_j, r, a)
                ideal_inter(contains_i, contains_j, r * a)
            }
        }
    }
}

/// The intersection of two ideals is an ideal.
theorem ideal_inter_is_ideal[R: CommRing](contains_i: R -> Bool,
                                          contains_j: R -> Bool) {
    is_ideal(contains_i) and is_ideal(contains_j)
        implies is_ideal(ideal_inter[R](contains_i, contains_j))
} by {
    if is_ideal(contains_i) and is_ideal(contains_j) {
        ideal_inter_zero_constraint(contains_i, contains_j)
        ideal_inter_add_constraint(contains_i, contains_j)
        ideal_inter_absorb_constraint(contains_i, contains_j)
        is_ideal_unfold(ideal_inter[R](contains_i, contains_j))
        is_ideal(ideal_inter[R](contains_i, contains_j))
    }
}

/// The intersection is contained in the left subset.
theorem ideal_inter_subset_left[R: CommRing](contains_i: R -> Bool,
                                             contains_j: R -> Bool) {
    ideal_subset(ideal_inter[R](contains_i, contains_j), contains_i)
} by {
    forall(x: R) {
        if ideal_inter(contains_i, contains_j, x) {
            contains_i(x)
        }
    }
}

/// The intersection is contained in the right subset.
theorem ideal_inter_subset_right[R: CommRing](contains_i: R -> Bool,
                                              contains_j: R -> Bool) {
    ideal_subset(ideal_inter[R](contains_i, contains_j), contains_j)
} by {
    forall(x: R) {
        if ideal_inter(contains_i, contains_j, x) {
            contains_j(x)
        }
    }
}

/// Any subset contained in both subsets is contained in their intersection.
theorem ideal_inter_glb[R: CommRing](contains_i: R -> Bool,
                                     contains_j: R -> Bool,
                                     contains_k: R -> Bool) {
    ideal_subset(contains_k, contains_i) and ideal_subset(contains_k, contains_j)
        implies ideal_subset(contains_k, ideal_inter[R](contains_i, contains_j))
} by {
    if ideal_subset(contains_k, contains_i) and ideal_subset(contains_k, contains_j) {
        forall(x: R) {
            if contains_k(x) {
                contains_i(x)
                contains_j(x)
                ideal_inter(contains_i, contains_j, x)
            }
        }
    }
}

/// Containment in an ideal intersection is equivalent to containment in both ideals.
theorem ideal_subset_inter_iff[R: CommRing](contains_k: R -> Bool,
                                           contains_i: R -> Bool,
                                           contains_j: R -> Bool) {
    ideal_subset(contains_k, ideal_inter[R](contains_i, contains_j)) =
        (ideal_subset(contains_k, contains_i) and ideal_subset(contains_k, contains_j))
} by {
    if ideal_subset(contains_k, ideal_inter[R](contains_i, contains_j)) {
        ideal_inter_subset_left(contains_i, contains_j)
        ideal_subset_trans(contains_k, ideal_inter[R](contains_i, contains_j), contains_i)
        ideal_inter_subset_right(contains_i, contains_j)
        ideal_subset_trans(contains_k, ideal_inter[R](contains_i, contains_j), contains_j)
        ideal_subset(contains_k, contains_j)
        ideal_subset(contains_k, contains_i) and ideal_subset(contains_k, contains_j)
    }
    if ideal_subset(contains_k, contains_i) and ideal_subset(contains_k, contains_j) {
        ideal_inter_glb(contains_i, contains_j, contains_k)
        ideal_subset(contains_k, ideal_inter[R](contains_i, contains_j))
    }
}

/// The intersection is commutative.
theorem ideal_inter_comm[R: CommRing](contains_i: R -> Bool,
                                      contains_j: R -> Bool) {
    ideal_inter[R](contains_i, contains_j) = ideal_inter[R](contains_j, contains_i)
} by {
    forall(x: R) {
        ideal_inter[R](contains_i, contains_j)(x) = (contains_i(x) and contains_j(x))
        ideal_inter[R](contains_j, contains_i)(x) = (contains_j(x) and contains_i(x))
        ideal_inter[R](contains_i, contains_j)(x) = ideal_inter[R](contains_j, contains_i)(x)
    }
}

/// The intersection of ideals is associative.
theorem ideal_inter_assoc[R: CommRing](contains_i: R -> Bool,
                                       contains_j: R -> Bool,
                                       contains_k: R -> Bool) {
    ideal_inter[R](contains_i, ideal_inter[R](contains_j, contains_k)) =
        ideal_inter[R](ideal_inter[R](contains_i, contains_j), contains_k)
} by {
    forall(x: R) {
        if ideal_inter[R](contains_i, ideal_inter[R](contains_j, contains_k))(x) {
            ideal_inter[R](contains_j, contains_k)(x)
            contains_j(x)
            ideal_inter[R](contains_i, contains_j)(x)
            ideal_inter[R](ideal_inter[R](contains_i, contains_j), contains_k)(x)
        }
        if ideal_inter[R](ideal_inter[R](contains_i, contains_j), contains_k)(x) {
            ideal_inter[R](contains_i, contains_j)(x)
            contains_j(x)
            ideal_inter[R](contains_j, contains_k)(x)
            ideal_inter[R](contains_i, ideal_inter[R](contains_j, contains_k))(x)
        }
        ideal_inter[R](contains_i, ideal_inter[R](contains_j, contains_k))(x) =
            ideal_inter[R](ideal_inter[R](contains_i, contains_j), contains_k)(x)
    }
}

/// The intersection of an ideal with itself is the same ideal.
theorem ideal_inter_idempotent[R: CommRing](contains: R -> Bool) {
    ideal_inter[R](contains, contains) = contains
} by {
    forall(x: R) {
        if ideal_inter[R](contains, contains)(x) {
            contains(x)
        }
        if contains(x) {
            ideal_inter[R](contains, contains)(x)
        }
        ideal_inter[R](contains, contains)(x) = contains(x)
    }
}

/// Intersecting with a larger ideal gives the smaller ideal.
theorem ideal_inter_eq_left_of_subset[R: CommRing](contains_i: R -> Bool,
                                                   contains_j: R -> Bool) {
    ideal_subset(contains_i, contains_j) implies ideal_inter[R](contains_i, contains_j) = contains_i
} by {
    if ideal_subset(contains_i, contains_j) {
        ideal_inter_subset_left(contains_i, contains_j)
        ideal_inter_glb(contains_i, contains_j, contains_i)
        ideal_subset(contains_i, ideal_inter[R](contains_i, contains_j))
        ideal_subset_antisymm(ideal_inter[R](contains_i, contains_j), contains_i)
        ideal_inter[R](contains_i, contains_j) = contains_i
    }
}

/// Intersecting with a larger ideal on the left gives the smaller ideal.
theorem ideal_inter_eq_right_of_subset[R: CommRing](contains_i: R -> Bool,
                                                    contains_j: R -> Bool) {
    ideal_subset(contains_j, contains_i) implies ideal_inter[R](contains_i, contains_j) = contains_j
} by {
    if ideal_subset(contains_j, contains_i) {
        ideal_inter_subset_right(contains_i, contains_j)
        ideal_inter_glb(contains_i, contains_j, contains_j)
        ideal_subset(contains_j, ideal_inter[R](contains_i, contains_j))
        ideal_subset_antisymm(ideal_inter[R](contains_i, contains_j), contains_j)
        ideal_inter[R](contains_i, contains_j) = contains_j
    }
}

/// Intersecting the zero ideal on the left gives the zero ideal.
theorem zero_ideal_inter_left[R: CommRing](contains: R -> Bool) {
    is_ideal(contains) implies ideal_inter[R](zero_ideal[R], contains) = zero_ideal[R]
} by {
    if is_ideal(contains) {
        zero_ideal_subset(contains)
        ideal_inter_eq_left_of_subset(zero_ideal[R], contains)
    }
}

/// Intersecting the zero ideal on the right gives the zero ideal.
theorem zero_ideal_inter_right[R: CommRing](contains: R -> Bool) {
    is_ideal(contains) implies ideal_inter[R](contains, zero_ideal[R]) = zero_ideal[R]
} by {
    if is_ideal(contains) {
        zero_ideal_subset(contains)
        ideal_inter_eq_right_of_subset(contains, zero_ideal[R])
    }
}

/// Intersecting the unit ideal on the left gives the other ideal.
theorem unit_ideal_inter_left[R: CommRing](contains: R -> Bool) {
    ideal_inter[R](unit_ideal[R], contains) = contains
} by {
    subset_unit_ideal(contains)
    ideal_inter_eq_right_of_subset(unit_ideal[R], contains)
}

/// Intersecting the unit ideal on the right gives the other ideal.
theorem unit_ideal_inter_right[R: CommRing](contains: R -> Bool) {
    ideal_inter[R](contains, unit_ideal[R]) = contains
} by {
    subset_unit_ideal(contains)
    ideal_inter_eq_left_of_subset(contains, unit_ideal[R])
}

/// Membership in the sum of two subsets: elements decomposable as a + b
/// with a in the first subset and b in the second.
define ideal_sum[R: CommRing](contains_i: R -> Bool,
                              contains_j: R -> Bool,
                              x: R) -> Bool {
    exists(a: R, b: R) {
        contains_i(a) and contains_j(b) and x = a + b
    }
}

/// The sum of two ideals contains the additive identity.
theorem ideal_sum_zero_constraint[R: CommRing](contains_i: R -> Bool,
                                               contains_j: R -> Bool) {
    is_ideal(contains_i) and is_ideal(contains_j)
        implies ideal_zero_constraint(ideal_sum[R](contains_i, contains_j))
} by {
    if is_ideal(contains_i) and is_ideal(contains_j) {
        is_ideal_zero(contains_i)
        is_ideal_zero(contains_j)
        ideal_sum(contains_i, contains_j, R.0)
    }
}

/// The sum of two ideals is closed under addition.
theorem ideal_sum_add_constraint[R: CommRing](contains_i: R -> Bool,
                                              contains_j: R -> Bool) {
    is_ideal(contains_i) and is_ideal(contains_j)
        implies ideal_add_constraint(ideal_sum[R](contains_i, contains_j))
} by {
    if is_ideal(contains_i) and is_ideal(contains_j) {
        forall(x: R, y: R) {
            if ideal_sum(contains_i, contains_j, x) and ideal_sum(contains_i, contains_j, y) {
                let (ax: R, bx: R) satisfy {
                    contains_i(ax) and contains_j(bx) and x = ax + bx
                }
                let (ay: R, b2: R) satisfy {
                    contains_i(ay) and contains_j(b2) and y = ay + b2
                }
                is_ideal_add(contains_i, ax, ay)
                is_ideal_add(contains_j, bx, b2)
                contains_i(ax + ay)
                contains_j(bx + b2)
                x + y = (ax + bx) + (ay + b2)
                (ax + bx) + (ay + b2) = (ax + ay) + (bx + b2)
                ideal_sum(contains_i, contains_j, x + y)
            }
        }
    }
}

/// The sum of two ideals absorbs ring multiplication.
theorem ideal_sum_absorb_constraint[R: CommRing](contains_i: R -> Bool,
                                                 contains_j: R -> Bool) {
    is_ideal(contains_i) and is_ideal(contains_j)
        implies ideal_absorb_constraint(ideal_sum[R](contains_i, contains_j))
} by {
    if is_ideal(contains_i) and is_ideal(contains_j) {
        forall(r: R, x: R) {
            if ideal_sum(contains_i, contains_j, x) {
                let (a: R, b: R) satisfy {
                    contains_i(a) and contains_j(b) and x = a + b
                }
                is_ideal_absorb(contains_i, r, a)
                is_ideal_absorb(contains_j, r, b)
                ideal_sum(contains_i, contains_j, r * x)
            }
        }
    }
}

/// The sum of two ideals is an ideal.
theorem ideal_sum_is_ideal[R: CommRing](contains_i: R -> Bool,
                                        contains_j: R -> Bool) {
    is_ideal(contains_i) and is_ideal(contains_j)
        implies is_ideal(ideal_sum[R](contains_i, contains_j))
} by {
    if is_ideal(contains_i) and is_ideal(contains_j) {
        ideal_sum_zero_constraint(contains_i, contains_j)
        ideal_sum_add_constraint(contains_i, contains_j)
        ideal_sum_absorb_constraint(contains_i, contains_j)
        is_ideal_unfold(ideal_sum[R](contains_i, contains_j))
        is_ideal(ideal_sum[R](contains_i, contains_j))
    }
}

/// The first ideal is contained in the sum of two ideals.
theorem ideal_sum_subset_left[R: CommRing](contains_i: R -> Bool,
                                           contains_j: R -> Bool) {
    is_ideal(contains_j) implies ideal_subset(contains_i, ideal_sum[R](contains_i, contains_j))
} by {
    if is_ideal(contains_j) {
        is_ideal_zero(contains_j)
        forall(x: R) {
            if contains_i(x) {
                ideal_sum(contains_i, contains_j, x)
            }
        }
    }
}

/// The second ideal is contained in the sum of two ideals.
theorem ideal_sum_subset_right[R: CommRing](contains_i: R -> Bool,
                                            contains_j: R -> Bool) {
    is_ideal(contains_i) implies ideal_subset(contains_j, ideal_sum[R](contains_i, contains_j))
} by {
    if is_ideal(contains_i) {
        is_ideal_zero(contains_i)
        forall(x: R) {
            if contains_j(x) {
                ideal_sum(contains_i, contains_j, x)
            }
        }
    }
}

/// Any ideal containing both subsets contains their sum.
theorem ideal_sum_lub[R: CommRing](contains_i: R -> Bool,
                                   contains_j: R -> Bool,
                                   contains_k: R -> Bool) {
    is_ideal(contains_k) and ideal_subset(contains_i, contains_k) and ideal_subset(contains_j, contains_k)
        implies ideal_subset(ideal_sum[R](contains_i, contains_j), contains_k)
} by {
    if is_ideal(contains_k) and ideal_subset(contains_i, contains_k) and ideal_subset(contains_j, contains_k) {
        forall(x: R) {
            if ideal_sum(contains_i, contains_j, x) {
                let (a: R, b: R) satisfy {
                    contains_i(a) and contains_j(b) and x = a + b
                }
                contains_k(a)
                contains_k(b)
                is_ideal_add(contains_k, a, b)
                contains_k(x)
            }
        }
    }
}

/// Containment of an ideal sum is equivalent to containment of both summands.
theorem ideal_sum_subset_iff[R: CommRing](contains_i: R -> Bool,
                                          contains_j: R -> Bool,
                                          contains_k: R -> Bool) {
    is_ideal(contains_i) and is_ideal(contains_j) and is_ideal(contains_k)
        implies
    (ideal_subset(ideal_sum[R](contains_i, contains_j), contains_k) =
        (ideal_subset(contains_i, contains_k) and ideal_subset(contains_j, contains_k)))
} by {
    if is_ideal(contains_i) and is_ideal(contains_j) and is_ideal(contains_k) {
        if ideal_subset(ideal_sum[R](contains_i, contains_j), contains_k) {
            ideal_sum_subset_left(contains_i, contains_j)
            ideal_subset_trans(contains_i, ideal_sum[R](contains_i, contains_j), contains_k)
            ideal_sum_subset_right(contains_i, contains_j)
            ideal_subset_trans(contains_j, ideal_sum[R](contains_i, contains_j), contains_k)
            ideal_subset(contains_j, contains_k)
            ideal_subset(contains_i, contains_k) and ideal_subset(contains_j, contains_k)
        }
        if ideal_subset(contains_i, contains_k) and ideal_subset(contains_j, contains_k) {
            ideal_sum_lub(contains_i, contains_j, contains_k)
            ideal_subset(ideal_sum[R](contains_i, contains_j), contains_k)
        }
        ideal_subset(ideal_sum[R](contains_i, contains_j), contains_k) =
            (ideal_subset(contains_i, contains_k) and ideal_subset(contains_j, contains_k))
    }
}

/// The sum is commutative.
theorem ideal_sum_comm[R: CommRing](contains_i: R -> Bool,
                                    contains_j: R -> Bool) {
    ideal_sum[R](contains_i, contains_j) = ideal_sum[R](contains_j, contains_i)
} by {
    forall(x: R) {
        if ideal_sum(contains_i, contains_j, x) {
            let (a: R, b: R) satisfy {
                contains_i(a) and contains_j(b) and x = a + b
            }
            ideal_sum(contains_j, contains_i, x)
        }
        if ideal_sum(contains_j, contains_i, x) {
            let (a: R, b: R) satisfy {
                contains_j(a) and contains_i(b) and x = a + b
            }
            ideal_sum(contains_i, contains_j, x)
        }
        ideal_sum[R](contains_i, contains_j)(x) = ideal_sum[R](contains_j, contains_i)(x)
    }
}

/// The sum of ideals is associative.
theorem ideal_sum_assoc[R: CommRing](contains_i: R -> Bool,
                                     contains_j: R -> Bool,
                                     contains_k: R -> Bool) {
    ideal_sum[R](contains_i, ideal_sum[R](contains_j, contains_k)) =
        ideal_sum[R](ideal_sum[R](contains_i, contains_j), contains_k)
} by {
    forall(x: R) {
        if ideal_sum[R](contains_i, ideal_sum[R](contains_j, contains_k))(x) {
            let (a: R, bc: R) satisfy {
                contains_i(a) and ideal_sum[R](contains_j, contains_k)(bc) and x = a + bc
            }
            let (b: R, c: R) satisfy {
                contains_j(b) and contains_k(c) and bc = b + c
            }
            ideal_sum[R](contains_i, contains_j)(a + b)
            exists(u: R, v: R) {
                ideal_sum[R](contains_i, contains_j)(u) and contains_k(v) and x = u + v
            }
            ideal_sum[R](ideal_sum[R](contains_i, contains_j), contains_k)(x)
        }
        if ideal_sum[R](ideal_sum[R](contains_i, contains_j), contains_k)(x) {
            let (ab: R, c: R) satisfy {
                ideal_sum[R](contains_i, contains_j)(ab) and contains_k(c) and x = ab + c
            }
            let (a: R, b: R) satisfy {
                contains_i(a) and contains_j(b) and ab = a + b
            }
            ideal_sum[R](contains_j, contains_k)(b + c)
            exists(u: R, v: R) {
                contains_i(u) and ideal_sum[R](contains_j, contains_k)(v) and x = u + v
            }
            ideal_sum[R](contains_i, ideal_sum[R](contains_j, contains_k))(x)
        }
        ideal_sum[R](contains_i, ideal_sum[R](contains_j, contains_k))(x) =
            ideal_sum[R](ideal_sum[R](contains_i, contains_j), contains_k)(x)
    }
}

/// The sum of an ideal with itself is the same ideal.
theorem ideal_sum_idempotent[R: CommRing](contains: R -> Bool) {
    is_ideal(contains) implies ideal_sum[R](contains, contains) = contains
} by {
    if is_ideal(contains) {
        ideal_subset_refl(contains)
        ideal_sum_lub(contains, contains, contains)
        ideal_sum_subset_left(contains, contains)
        ideal_subset_antisymm(ideal_sum[R](contains, contains), contains)
        ideal_sum[R](contains, contains) = contains
    }
}

/// Adding a contained ideal on the right gives the larger ideal.
theorem ideal_sum_eq_left_of_subset[R: CommRing](contains_i: R -> Bool,
                                                 contains_j: R -> Bool) {
    is_ideal(contains_i) and is_ideal(contains_j) and ideal_subset(contains_j, contains_i)
        implies ideal_sum[R](contains_i, contains_j) = contains_i
} by {
    if is_ideal(contains_i) and is_ideal(contains_j) and ideal_subset(contains_j, contains_i) {
        ideal_subset_refl(contains_i)
        ideal_sum_lub(contains_i, contains_j, contains_i)
        ideal_sum_subset_left(contains_i, contains_j)
        ideal_subset_antisymm(ideal_sum[R](contains_i, contains_j), contains_i)
        ideal_sum[R](contains_i, contains_j) = contains_i
    }
}

/// Adding a contained ideal on the left gives the larger ideal.
theorem ideal_sum_eq_right_of_subset[R: CommRing](contains_i: R -> Bool,
                                                  contains_j: R -> Bool) {
    is_ideal(contains_i) and is_ideal(contains_j) and ideal_subset(contains_i, contains_j)
        implies ideal_sum[R](contains_i, contains_j) = contains_j
} by {
    if is_ideal(contains_i) and is_ideal(contains_j) and ideal_subset(contains_i, contains_j) {
        ideal_subset_refl(contains_j)
        ideal_sum_lub(contains_i, contains_j, contains_j)
        ideal_sum_subset_right(contains_i, contains_j)
        ideal_subset_antisymm(ideal_sum[R](contains_i, contains_j), contains_j)
        ideal_sum[R](contains_i, contains_j) = contains_j
    }
}

/// Adding the zero ideal on the left gives the other ideal.
theorem zero_ideal_sum_left[R: CommRing](contains: R -> Bool) {
    is_ideal(contains) implies ideal_sum[R](zero_ideal[R], contains) = contains
} by {
    if is_ideal(contains) {
        zero_ideal_is_ideal[R]
        zero_ideal_subset(contains)
        ideal_sum_eq_right_of_subset(zero_ideal[R], contains)
    }
}

/// Adding the zero ideal on the right gives the other ideal.
theorem zero_ideal_sum_right[R: CommRing](contains: R -> Bool) {
    is_ideal(contains) implies ideal_sum[R](contains, zero_ideal[R]) = contains
} by {
    if is_ideal(contains) {
        zero_ideal_is_ideal[R]
        zero_ideal_subset(contains)
        ideal_sum_eq_left_of_subset(contains, zero_ideal[R])
    }
}

/// Adding the unit ideal on the left gives the unit ideal.
theorem unit_ideal_sum_left[R: CommRing](contains: R -> Bool) {
    is_ideal(contains) implies ideal_sum[R](unit_ideal[R], contains) = unit_ideal[R]
} by {
    if is_ideal(contains) {
        unit_ideal_is_ideal[R]
        subset_unit_ideal(contains)
        ideal_sum_eq_left_of_subset(unit_ideal[R], contains)
    }
}

/// Adding the unit ideal on the right gives the unit ideal.
theorem unit_ideal_sum_right[R: CommRing](contains: R -> Bool) {
    is_ideal(contains) implies ideal_sum[R](contains, unit_ideal[R]) = unit_ideal[R]
} by {
    if is_ideal(contains) {
        unit_ideal_is_ideal[R]
        subset_unit_ideal(contains)
        ideal_sum_eq_right_of_subset(contains, unit_ideal[R])
    }
}

/// An ideal of a commutative ring, represented as a subset closed under addition and absorbing multiplication.
structure Ideal[R: CommRing] {
    /// True if the given element is a member of this ideal.
    contains: R -> Bool
} constraint {
    is_ideal(contains)
}

/// Ideal extensionality from equality of membership.
theorem ideal_ext[R: CommRing](a: Ideal[R], b: Ideal[R]) {
    (forall(x: R) { a.contains(x) = b.contains(x) }) implies a = b
} by {
    if forall(x: R) { a.contains(x) = b.contains(x) } {
        a.contains = b.contains
    }
}

/// Equal ideals have equal membership predicates.
theorem ideal_eq_contains[R: CommRing](a: Ideal[R], b: Ideal[R]) {
    a = b implies a.contains = b.contains
}

/// Equal ideals have equal membership at every element.
theorem ideal_eq_contains_at[R: CommRing](a: Ideal[R], b: Ideal[R], x: R) {
    a = b implies a.contains(x) = b.contains(x)
} by {
    if a = b {
        a.contains(x) = b.contains(x)
    }
}

/// Equality of membership predicates determines equality of ideals.
theorem ideal_eq_of_contains_eq[R: CommRing](a: Ideal[R], b: Ideal[R]) {
    a.contains = b.contains implies a = b
} by {
    if a.contains = b.contains {
        forall(x: R) {
            a.contains(x) = b.contains(x)
        }
        ideal_ext(a, b)
    }
}

/// Pointwise equality of membership determines equality of ideals.
theorem ideal_eq_of_contains_at_eq[R: CommRing](a: Ideal[R], b: Ideal[R]) {
    (forall(x: R) { a.contains(x) = b.contains(x) }) implies a = b
} by {
    if forall(x: R) { a.contains(x) = b.contains(x) } {
        ideal_ext(a, b)
    }
}

/// Membership of zero in any ideal.
theorem ideal_contains_zero[R: CommRing](i: Ideal[R]) {
    i.contains(R.0)
} by {
    is_ideal_zero(i.contains)
}

/// Closure of an ideal under addition.
theorem ideal_contains_add[R: CommRing](i: Ideal[R], a: R, b: R) {
    i.contains(a) and i.contains(b) implies i.contains(a + b)
} by {
    if i.contains(a) and i.contains(b) {
        is_ideal_add(i.contains, a, b)
        i.contains(a + b)
    }
}

/// Absorption of an ideal by multiplication.
theorem ideal_contains_mul_left[R: CommRing](i: Ideal[R], r: R, a: R) {
    i.contains(a) implies i.contains(r * a)
} by {
    if i.contains(a) {
        is_ideal_absorb(i.contains, r, a)
        i.contains(r * a)
    }
}

/// Absorption of an ideal by multiplication on the right.
theorem ideal_contains_mul_right[R: CommRing](i: Ideal[R], a: R, r: R) {
    i.contains(a) implies i.contains(a * r)
} by {
    if i.contains(a) {
        ideal_contains_mul_left(i, r, a)
        r * a = a * r
    }
}

/// Closure of an ideal under negation.
theorem ideal_contains_neg[R: CommRing](i: Ideal[R], a: R) {
    i.contains(a) implies i.contains(-a)
} by {
    if i.contains(a) {
        is_ideal_neg(i.contains, a)
        i.contains(-a)
    }
}

/// The bundled zero ideal.
let bundled_zero_ideal[R: CommRing]: Ideal[R] satisfy {
    Ideal.new(zero_ideal[R]) = Option.some(bundled_zero_ideal)
}

/// The bundled unit ideal.
let bundled_unit_ideal[R: CommRing]: Ideal[R] satisfy {
    Ideal.new(unit_ideal[R]) = Option.some(bundled_unit_ideal)
}

/// The bundled principal ideal generated by an element.
let bundled_principal_ideal[R: CommRing](a: R) -> result: Ideal[R] satisfy {
    Ideal.new(principal_ideal[R](a)) = Option.some(result)
} by {
    principal_ideal_is_ideal(a)
}

attributes Ideal[R: CommRing] {
    /// Ideal extensionality from pointwise equality of membership.
    let ext = ideal_ext[R]

    /// The subset of ring elements belonging to this ideal.
    define as_set(self) -> Set[R] {
        Set[R].new(self.contains)
    }

    /// The zero ideal.
    let zero = bundled_zero_ideal[R]

    /// The unit ideal.
    let unit = bundled_unit_ideal[R]

    /// The principal ideal generated by an element.
    let principal: R -> Ideal[R] = bundled_principal_ideal
}

/// Membership in the underlying set is membership in the ideal.
theorem ideal_as_set_contains_eq[R: CommRing](i: Ideal[R], x: R) {
    i.as_set.contains(x) = i.contains(x)
} by {
}

/// Membership in an ideal is membership in its underlying set.
theorem ideal_contains_as_set_eq[R: CommRing](i: Ideal[R], x: R) {
    i.contains(x) = i.as_set.contains(x)
} by {
    ideal_as_set_contains_eq(i, x)
}

/// Equal ideals have equal underlying sets.
theorem ideal_eq_as_set[R: CommRing](a: Ideal[R], b: Ideal[R]) {
    a = b implies a.as_set = b.as_set
}

/// Membership in the bundled zero ideal is equality to zero.
theorem bundled_zero_ideal_contains_eq[R: CommRing](x: R) {
    Ideal[R].zero.contains(x) = (x = R.0)
} by {
    Ideal[R].zero.contains(x) = zero_ideal[R](x)
}

/// The bundled zero ideal contains only zero.
theorem bundled_zero_ideal_only_has_zero[R: CommRing](x: R) {
    Ideal[R].zero.contains(x) implies x = R.0
} by {
    if Ideal[R].zero.contains(x) {
        bundled_zero_ideal_contains_eq(x)
        x = R.0
    }
}

/// Every element belongs to the bundled unit ideal.
theorem bundled_unit_ideal_contains_everything[R: CommRing](x: R) {
    Ideal[R].unit.contains(x)
} by {
    Ideal[R].unit.contains(x) = unit_ideal[R](x)
}

/// Membership in the bundled unit ideal is always true.
theorem bundled_unit_ideal_contains_eq[R: CommRing](x: R) {
    Ideal[R].unit.contains(x) = true
} by {
    bundled_unit_ideal_contains_everything(x)
}

/// A bundled ideal that contains the multiplicative identity is the bundled unit ideal.
theorem bundled_ideal_eq_unit_of_contains_one[R: CommRing](i: Ideal[R]) {
    i.contains(R.1) implies i = Ideal[R].unit
} by {
    if i.contains(R.1) {
        ideal_eq_unit_of_contains_one(i.contains)
        forall(x: R) {
            bundled_unit_ideal_contains_eq(x)
            unit_ideal[R](x) = Ideal[R].unit.contains(x)
            i.contains(x) = Ideal[R].unit.contains(x)
        }
        ideal_ext(i, Ideal[R].unit)
    }
}

/// A bundled ideal contains the multiplicative identity exactly when it is the bundled unit ideal.
theorem bundled_ideal_contains_one_iff_eq_unit[R: CommRing](i: Ideal[R]) {
    i.contains(R.1) = (i = Ideal[R].unit)
} by {
    if i.contains(R.1) {
        bundled_ideal_eq_unit_of_contains_one(i)
        i = Ideal[R].unit
    }
    if i = Ideal[R].unit {
        bundled_unit_ideal_contains_everything(R.1)
        i.contains(R.1)
    }
}

/// Membership in a bundled principal ideal is divisibility by its generator.
theorem bundled_principal_ideal_contains_eq[R: CommRing](a: R, x: R) {
    Ideal[R].principal(a).contains(x) = principal_ideal(a, x)
} by {
    Ideal[R].principal(a).contains(x) = principal_ideal(a, x)
}

/// The generator belongs to its bundled principal ideal.
theorem bundled_principal_ideal_contains_generator[R: CommRing](a: R) {
    Ideal[R].principal(a).contains(a)
} by {
    principal_ideal_contains_generator(a)
    bundled_principal_ideal_contains_eq(a, a)
}

/// The bundled principal ideal generated by zero is the bundled zero ideal.
theorem bundled_principal_ideal_zero_eq_zero[R: CommRing] {
    Ideal[R].principal(R.0) = Ideal[R].zero
} by {
    forall(x: R) {
        bundled_principal_ideal_contains_eq(R.0, x)
        bundled_zero_ideal_contains_eq(x)
        principal_ideal_zero_eq_zero_ideal[R]
        Ideal[R].principal(R.0).contains(x) = Ideal[R].zero.contains(x)
    }
    ideal_ext(Ideal[R].principal(R.0), Ideal[R].zero)
}

/// The bundled principal ideal generated by one is the bundled unit ideal.
theorem bundled_principal_ideal_one_eq_unit[R: CommRing] {
    Ideal[R].principal(R.1) = Ideal[R].unit
} by {
    forall(x: R) {
        bundled_principal_ideal_contains_eq(R.1, x)
        bundled_unit_ideal_contains_eq(x)
        principal_ideal_one_eq_unit_ideal[R]
        principal_ideal[R](R.1)(x) = unit_ideal[R](x)
        Ideal[R].principal(R.1).contains(x) = Ideal[R].unit.contains(x)
    }
    ideal_ext(Ideal[R].principal(R.1), Ideal[R].unit)
}

/// The bundled principal ideal contains every multiple of its generator.
theorem bundled_principal_ideal_contains_multiple[R: CommRing](a: R, r: R) {
    Ideal[R].principal(a).contains(r * a)
} by {
    principal_ideal_contains_multiple(a, r)
    bundled_principal_ideal_contains_eq(a, r * a)
}

/// Membership in a bundled principal ideal yields a witnessing multiple.
theorem bundled_principal_ideal_witness[R: CommRing](a: R, x: R) {
    Ideal[R].principal(a).contains(x) implies exists(r: R) { x = r * a }
} by {
    if Ideal[R].principal(a).contains(x) {
        bundled_principal_ideal_contains_eq(a, x)
        principal_ideal_witness(a, x)
        exists(r: R) { x = r * a }
    }
}

/// True if one bundled ideal is contained in another.
define bundled_ideal_subset[R: CommRing](a: Ideal[R], b: Ideal[R]) -> Bool {
    forall(x: R) {
        a.contains(x) implies b.contains(x)
    }
}

/// Bundled ideal containment agrees with predicate containment.
theorem bundled_ideal_subset_eq[R: CommRing](a: Ideal[R], b: Ideal[R]) {
    bundled_ideal_subset(a, b) = ideal_subset(a.contains, b.contains)
} by {
    if bundled_ideal_subset(a, b) {
        forall(x: R) {
            if a.contains(x) {
                b.contains(x)
            }
        }
        ideal_subset(a.contains, b.contains)
    }
    if ideal_subset(a.contains, b.contains) {
        forall(x: R) {
            if a.contains(x) {
                b.contains(x)
            }
        }
        bundled_ideal_subset(a, b)
    }
}

/// Bundled ideal containment agrees with containment of underlying sets.
theorem bundled_ideal_subset_as_set_eq[R: CommRing](a: Ideal[R], b: Ideal[R]) {
    bundled_ideal_subset(a, b) = a.as_set.subset(b.as_set)
} by {
    if bundled_ideal_subset(a, b) {
        forall(x: R) {
            if a.as_set.contains(x) {
                ideal_as_set_contains_eq(a, x)
                bundled_ideal_subset(a, b) = forall(y: R) {
                    a.contains(y) implies b.contains(y)
                }
                ideal_contains_as_set_eq(b, x)
                b.as_set.contains(x)
            }
        }
        a.as_set.subset(b.as_set)
    }
    if a.as_set.subset(b.as_set) {
        forall(x: R) {
            if a.contains(x) {
                ideal_contains_as_set_eq(a, x)
                a.as_set.subset(b.as_set) = forall(y: R) {
                    a.as_set.contains(y) implies b.as_set.contains(y)
                }
                ideal_as_set_contains_eq(b, x)
                b.contains(x)
            }
        }
        bundled_ideal_subset(a, b)
    }
}

/// Bundled ideal containment is reflexive.
theorem bundled_ideal_subset_refl[R: CommRing](a: Ideal[R]) {
    bundled_ideal_subset(a, a)
} by {
    forall(x: R) {
        if a.contains(x) {
            a.contains(x)
        }
    }
}

/// Bundled ideal containment is transitive.
theorem bundled_ideal_subset_trans[R: CommRing](a: Ideal[R], b: Ideal[R], c: Ideal[R]) {
    bundled_ideal_subset(a, b) and bundled_ideal_subset(b, c) implies bundled_ideal_subset(a, c)
} by {
    if bundled_ideal_subset(a, b) and bundled_ideal_subset(b, c) {
        bundled_ideal_subset_eq(a, b)
        ideal_subset(a.contains, b.contains)
        bundled_ideal_subset_eq(b, c)
        ideal_subset(b.contains, c.contains)
        forall(x: R) {
            if a.contains(x) {
                bundled_ideal_subset(a, b) = forall(y: R) {
                    a.contains(y) implies b.contains(y)
                }
                forall(y: R) {
                    a.contains(y) implies b.contains(y)
                }
                a.contains(x) implies b.contains(x)
                b.contains(x)
                c.contains(x)
            }
        }
    }
}

/// Mutual bundled ideal containment gives equality.
theorem bundled_ideal_subset_antisymm[R: CommRing](a: Ideal[R], b: Ideal[R]) {
    bundled_ideal_subset(a, b) and bundled_ideal_subset(b, a) implies a = b
} by {
    if bundled_ideal_subset(a, b) and bundled_ideal_subset(b, a) {
        bundled_ideal_subset_eq(a, b)
        bundled_ideal_subset_eq(b, a)
        ideal_subset_antisymm(a.contains, b.contains)
        ideal_eq_of_contains_eq(a, b)
        a = b
    }
}

/// Bundled ideal equality is mutual containment.
theorem bundled_ideal_eq_iff_subset_both[R: CommRing](a: Ideal[R], b: Ideal[R]) {
    a = b = (bundled_ideal_subset(a, b) and bundled_ideal_subset(b, a))
} by {
    if a = b {
        bundled_ideal_subset_refl(a)
        bundled_ideal_subset(b, a)
        bundled_ideal_subset(a, b) and bundled_ideal_subset(b, a)
    }
    if bundled_ideal_subset(a, b) and bundled_ideal_subset(b, a) {
        bundled_ideal_subset_antisymm(a, b)
        a = b
    }
}

/// The bundled zero ideal is contained in every ideal.
theorem bundled_zero_ideal_subset[R: CommRing](i: Ideal[R]) {
    bundled_ideal_subset(Ideal[R].zero, i)
} by {
    forall(x: R) {
        if Ideal[R].zero.contains(x) {
            bundled_zero_ideal_only_has_zero(x)
            ideal_contains_zero(i)
            i.contains(x)
        }
    }
}

/// Every ideal is contained in the bundled unit ideal.
theorem bundled_ideal_subset_unit[R: CommRing](i: Ideal[R]) {
    bundled_ideal_subset(i, Ideal[R].unit)
} by {
    forall(x: R) {
        if i.contains(x) {
            bundled_unit_ideal_contains_everything(x)
        }
    }
}

/// The bundled principal ideal of `a` is contained in any ideal containing `a`.
theorem bundled_principal_ideal_subset_of_contains_generator[R: CommRing](a: R, b: Ideal[R]) {
    b.contains(a) implies bundled_ideal_subset(Ideal[R].principal(a), b)
} by {
    if b.contains(a) {
        forall(x: R) {
            if Ideal[R].principal(a).contains(x) {
                bundled_principal_ideal_contains_eq(a, x)
                let r: R satisfy { x = r * a }
                is_ideal_absorb(b.contains, r, a)
                b.contains(x)
            }
        }
    }
}

/// A bundled principal ideal is contained in another iff its generator belongs to the other.
theorem bundled_principal_ideal_subset_iff_contains_generator[R: CommRing](a: R, b: Ideal[R]) {
    bundled_ideal_subset(Ideal[R].principal(a), b) = b.contains(a)
} by {
    if bundled_ideal_subset(Ideal[R].principal(a), b) {
        bundled_principal_ideal_contains_generator(a)
        bundled_ideal_subset(Ideal[R].principal(a), b) = forall(y: R) {
            Ideal[R].principal(a).contains(y) implies b.contains(y)
        }
        b.contains(a)
    }
    if b.contains(a) {
        bundled_principal_ideal_subset_of_contains_generator(a, b)
        bundled_ideal_subset(Ideal[R].principal(a), b)
    }
}

/// The intersection membership predicate of two bundled ideals.
define bundled_ideal_inter_contains[R: CommRing](a: Ideal[R], b: Ideal[R], x: R) -> Bool {
    a.contains(x) and b.contains(x)
}

/// The intersection of two bundled ideals is an ideal.
theorem bundled_ideal_inter_is_ideal[R: CommRing](a: Ideal[R], b: Ideal[R]) {
    is_ideal(bundled_ideal_inter_contains(a, b))
} by {
    ideal_inter_is_ideal(a.contains, b.contains)
    ideal_inter[R](a.contains, b.contains) = bundled_ideal_inter_contains(a, b)
}

/// The bundled intersection of two ideals.
let bundled_ideal_inter[R: CommRing](a: Ideal[R], b: Ideal[R]) -> result: Ideal[R] satisfy {
    Ideal.new(bundled_ideal_inter_contains(a, b)) = Option.some(result)
} by {
    bundled_ideal_inter_is_ideal(a, b)
}

/// The sum membership predicate of two bundled ideals.
define bundled_ideal_sum_contains[R: CommRing](a: Ideal[R], b: Ideal[R], x: R) -> Bool {
    exists(y: R, z: R) {
        a.contains(y) and b.contains(z) and x = y + z
    }
}

/// The sum of two bundled ideals is an ideal.
theorem bundled_ideal_sum_is_ideal[R: CommRing](a: Ideal[R], b: Ideal[R]) {
    is_ideal(bundled_ideal_sum_contains(a, b))
} by {
    ideal_sum_is_ideal(a.contains, b.contains)
    ideal_sum[R](a.contains, b.contains) = bundled_ideal_sum_contains(a, b)
}

/// The bundled sum of two ideals.
let bundled_ideal_sum[R: CommRing](a: Ideal[R], b: Ideal[R]) -> result: Ideal[R] satisfy {
    Ideal.new(bundled_ideal_sum_contains(a, b)) = Option.some(result)
} by {
    bundled_ideal_sum_is_ideal(a, b)
}

attributes Ideal[R: CommRing] {
    /// The common part of two ideals.
    let intersection: (Ideal[R], Ideal[R]) -> Ideal[R] = bundled_ideal_inter

    /// The least ideal containing two ideals.
    let sum: (Ideal[R], Ideal[R]) -> Ideal[R] = bundled_ideal_sum
}

/// Membership in the bundled intersection means membership in both ideals.
theorem bundled_ideal_inter_contains_eq[R: CommRing](a: Ideal[R], b: Ideal[R], x: R) {
    a.intersection(b).contains(x) = (a.contains(x) and b.contains(x))
} by {
    a.intersection(b).contains(x) = bundled_ideal_inter_contains(a, b, x)
}

/// The bundled intersection is contained in the left ideal.
theorem bundled_ideal_inter_subset_left[R: CommRing](a: Ideal[R], b: Ideal[R]) {
    bundled_ideal_subset(a.intersection(b), a)
} by {
    forall(x: R) {
        if a.intersection(b).contains(x) {
            bundled_ideal_inter_contains_eq(a, b, x)
            a.contains(x)
        }
    }
}

/// The bundled intersection is contained in the right ideal.
theorem bundled_ideal_inter_subset_right[R: CommRing](a: Ideal[R], b: Ideal[R]) {
    bundled_ideal_subset(a.intersection(b), b)
} by {
    forall(x: R) {
        if a.intersection(b).contains(x) {
            bundled_ideal_inter_contains_eq(a, b, x)
            b.contains(x)
        }
    }
}

/// Any ideal contained in two ideals is contained in their bundled intersection.
theorem bundled_ideal_subset_inter_of_subset_left_right[R: CommRing](
    c: Ideal[R],
    a: Ideal[R],
    b: Ideal[R]
) {
    bundled_ideal_subset(c, a) and bundled_ideal_subset(c, b) implies bundled_ideal_subset(c, a.intersection(b))
} by {
    if bundled_ideal_subset(c, a) and bundled_ideal_subset(c, b) {
        forall(x: R) {
            if c.contains(x) {
                a.contains(x)
                bundled_ideal_subset(c, b) = forall(y: R) {
                    c.contains(y) implies b.contains(y)
                }
                forall(y: R) {
                    c.contains(y) implies b.contains(y)
                }
                bundled_ideal_inter_contains_eq(a, b, x)
                a.intersection(b).contains(x)
            }
        }
    }
}

/// Containment in a bundled intersection is containment in both ideals.
theorem bundled_ideal_subset_inter_iff[R: CommRing](c: Ideal[R], a: Ideal[R], b: Ideal[R]) {
    bundled_ideal_subset(c, a.intersection(b)) =
        (bundled_ideal_subset(c, a) and bundled_ideal_subset(c, b))
} by {
    if bundled_ideal_subset(c, a.intersection(b)) {
        bundled_ideal_inter_subset_left(a, b)
        bundled_ideal_subset_trans(c, a.intersection(b), a)
        bundled_ideal_inter_subset_right(a, b)
        bundled_ideal_subset_trans(c, a.intersection(b), b)
        bundled_ideal_subset(c, b)
        bundled_ideal_subset(c, a) and bundled_ideal_subset(c, b)
    }
    if bundled_ideal_subset(c, a) and bundled_ideal_subset(c, b) {
        bundled_ideal_subset_inter_of_subset_left_right(c, a, b)
        bundled_ideal_subset(c, a.intersection(b))
    }
}

/// Bundled ideal intersection is commutative.
theorem bundled_ideal_inter_comm[R: CommRing](a: Ideal[R], b: Ideal[R]) {
    a.intersection(b) = b.intersection(a)
} by {
    forall(x: R) {
        if a.intersection(b).contains(x) {
            bundled_ideal_inter_contains_eq(a, b, x)
            bundled_ideal_inter_contains_eq(b, a, x)
            b.intersection(a).contains(x)
        }
        if b.intersection(a).contains(x) {
            bundled_ideal_inter_contains_eq(b, a, x)
            bundled_ideal_inter_contains_eq(a, b, x)
            a.intersection(b).contains(x)
        }
        a.intersection(b).contains(x) = b.intersection(a).contains(x)
    }
    ideal_ext(a.intersection(b), b.intersection(a))
}

/// Bundled ideal intersection is idempotent.
theorem bundled_ideal_inter_idempotent[R: CommRing](a: Ideal[R]) {
    a.intersection(a) = a
} by {
    forall(x: R) {
        if a.intersection(a).contains(x) {
            bundled_ideal_inter_contains_eq(a, a, x)
            a.contains(x)
        }
        if a.contains(x) {
            bundled_ideal_inter_contains_eq(a, a, x)
            a.intersection(a).contains(x)
        }
        a.intersection(a).contains(x) = a.contains(x)
    }
    ideal_ext(a.intersection(a), a)
}

/// Intersecting with a larger bundled ideal gives the smaller ideal.
theorem bundled_ideal_inter_eq_left_of_subset[R: CommRing](a: Ideal[R], b: Ideal[R]) {
    bundled_ideal_subset(a, b) implies a.intersection(b) = a
} by {
    if bundled_ideal_subset(a, b) {
        bundled_ideal_inter_subset_left(a, b)
        bundled_ideal_subset_inter_of_subset_left_right(a, a, b)
        bundled_ideal_subset(a, a.intersection(b))
        bundled_ideal_subset_antisymm(a.intersection(b), a)
        a.intersection(b) = a
    }
}

/// Intersecting with a larger bundled ideal on the left gives the smaller ideal.
theorem bundled_ideal_inter_eq_right_of_subset[R: CommRing](a: Ideal[R], b: Ideal[R]) {
    bundled_ideal_subset(b, a) implies a.intersection(b) = b
} by {
    if bundled_ideal_subset(b, a) {
        bundled_ideal_inter_subset_right(a, b)
        bundled_ideal_subset_inter_of_subset_left_right(b, a, b)
        bundled_ideal_subset_antisymm(a.intersection(b), b)
        a.intersection(b) = b
    }
}

/// Intersecting the bundled zero ideal on the left gives the zero ideal.
theorem bundled_zero_ideal_inter_left[R: CommRing](i: Ideal[R]) {
    Ideal[R].zero.intersection(i) = Ideal[R].zero
} by {
    bundled_zero_ideal_subset(i)
    bundled_ideal_inter_eq_left_of_subset(Ideal[R].zero, i)
}

/// Intersecting the bundled zero ideal on the right gives the zero ideal.
theorem bundled_zero_ideal_inter_right[R: CommRing](i: Ideal[R]) {
    i.intersection(Ideal[R].zero) = Ideal[R].zero
} by {
    bundled_zero_ideal_subset(i)
    bundled_ideal_inter_eq_right_of_subset(i, Ideal[R].zero)
}

/// Intersecting the bundled unit ideal on the left gives the other ideal.
theorem bundled_unit_ideal_inter_left[R: CommRing](i: Ideal[R]) {
    Ideal[R].unit.intersection(i) = i
} by {
    bundled_ideal_subset_unit(i)
    bundled_ideal_inter_eq_right_of_subset(Ideal[R].unit, i)
}

/// Intersecting the bundled unit ideal on the right gives the other ideal.
theorem bundled_unit_ideal_inter_right[R: CommRing](i: Ideal[R]) {
    i.intersection(Ideal[R].unit) = i
} by {
    bundled_ideal_subset_unit(i)
    bundled_ideal_inter_eq_left_of_subset(i, Ideal[R].unit)
}

/// Membership in the bundled sum is decomposition as a sum of elements from the summands.
theorem bundled_ideal_sum_contains_eq[R: CommRing](a: Ideal[R], b: Ideal[R], x: R) {
    a.sum(b).contains(x) = bundled_ideal_sum_contains(a, b, x)
} by {
    a.sum(b).contains(x) = bundled_ideal_sum_contains(a, b, x)
}

/// The left summand is contained in the bundled sum.
theorem bundled_ideal_sum_subset_left[R: CommRing](a: Ideal[R], b: Ideal[R]) {
    bundled_ideal_subset(a, a.sum(b))
} by {
    ideal_contains_zero(b)
    forall(x: R) {
        if a.contains(x) {
            bundled_ideal_sum_contains(a, b, x)
            bundled_ideal_sum_contains_eq(a, b, x)
            a.sum(b).contains(x)
        }
    }
}

/// The right summand is contained in the bundled sum.
theorem bundled_ideal_sum_subset_right[R: CommRing](a: Ideal[R], b: Ideal[R]) {
    bundled_ideal_subset(b, a.sum(b))
} by {
    ideal_contains_zero(a)
    forall(x: R) {
        if b.contains(x) {
            bundled_ideal_sum_contains(a, b, x)
            bundled_ideal_sum_contains_eq(a, b, x)
            a.sum(b).contains(x)
        }
    }
}

/// Any ideal containing both summands contains their bundled sum.
theorem bundled_ideal_sum_subset_of_left_right[R: CommRing](
    a: Ideal[R],
    b: Ideal[R],
    c: Ideal[R]
) {
    bundled_ideal_subset(a, c) and bundled_ideal_subset(b, c) implies bundled_ideal_subset(a.sum(b), c)
} by {
    if bundled_ideal_subset(a, c) and bundled_ideal_subset(b, c) {
        forall(x: R) {
            if a.sum(b).contains(x) {
                bundled_ideal_sum_contains_eq(a, b, x)
                bundled_ideal_sum_contains(a, b, x)
                let (y: R, z: R) satisfy {
                    a.contains(y) and b.contains(z) and x = y + z
                }
                c.contains(y)
                c.contains(z)
                ideal_contains_add(c, y, z)
                c.contains(x)
            }
        }
    }
}

/// Containment of a bundled sum is containment of both summands.
theorem bundled_ideal_sum_subset_iff[R: CommRing](a: Ideal[R], b: Ideal[R], c: Ideal[R]) {
    bundled_ideal_subset(a.sum(b), c) =
        (bundled_ideal_subset(a, c) and bundled_ideal_subset(b, c))
} by {
    if bundled_ideal_subset(a.sum(b), c) {
        bundled_ideal_sum_subset_left(a, b)
        bundled_ideal_subset_trans(a, a.sum(b), c)
        bundled_ideal_sum_subset_right(a, b)
        bundled_ideal_subset_trans(b, a.sum(b), c)
        bundled_ideal_subset(b, c)
        bundled_ideal_subset(a, c) and bundled_ideal_subset(b, c)
    }
    if bundled_ideal_subset(a, c) and bundled_ideal_subset(b, c) {
        bundled_ideal_sum_subset_of_left_right(a, b, c)
        bundled_ideal_subset(a.sum(b), c)
    }
}

/// Bundled ideal sum is commutative.
theorem bundled_ideal_sum_comm[R: CommRing](a: Ideal[R], b: Ideal[R]) {
    a.sum(b) = b.sum(a)
} by {
    forall(x: R) {
        if a.sum(b).contains(x) {
            bundled_ideal_sum_contains_eq(a, b, x)
            let (y: R, z: R) satisfy {
                a.contains(y) and b.contains(z) and x = y + z
            }
            bundled_ideal_sum_contains(b, a, x)
            bundled_ideal_sum_contains_eq(b, a, x)
            b.sum(a).contains(x)
        }
        if b.sum(a).contains(x) {
            bundled_ideal_sum_contains_eq(b, a, x)
            let (y: R, z: R) satisfy {
                b.contains(y) and a.contains(z) and x = y + z
            }
            bundled_ideal_sum_contains(a, b, x)
            bundled_ideal_sum_contains_eq(a, b, x)
            a.sum(b).contains(x)
        }
        a.sum(b).contains(x) = b.sum(a).contains(x)
    }
    ideal_ext(a.sum(b), b.sum(a))
}

/// Bundled ideal sum is idempotent.
theorem bundled_ideal_sum_idempotent[R: CommRing](a: Ideal[R]) {
    a.sum(a) = a
} by {
    bundled_ideal_subset_refl(a)
    bundled_ideal_sum_subset_of_left_right(a, a, a)
    bundled_ideal_sum_subset_left(a, a)
    bundled_ideal_subset_antisymm(a.sum(a), a)
}

/// Adding a contained bundled ideal on the right gives the larger ideal.
theorem bundled_ideal_sum_eq_left_of_subset[R: CommRing](a: Ideal[R], b: Ideal[R]) {
    bundled_ideal_subset(b, a) implies a.sum(b) = a
} by {
    if bundled_ideal_subset(b, a) {
        bundled_ideal_subset_refl(a)
        bundled_ideal_sum_subset_of_left_right(a, b, a)
        bundled_ideal_sum_subset_left(a, b)
        bundled_ideal_subset_antisymm(a.sum(b), a)
        a.sum(b) = a
    }
}

/// Adding a contained bundled ideal on the left gives the larger ideal.
theorem bundled_ideal_sum_eq_right_of_subset[R: CommRing](a: Ideal[R], b: Ideal[R]) {
    bundled_ideal_subset(a, b) implies a.sum(b) = b
} by {
    if bundled_ideal_subset(a, b) {
        bundled_ideal_subset_refl(b)
        bundled_ideal_sum_subset_of_left_right(a, b, b)
        bundled_ideal_sum_subset_right(a, b)
        bundled_ideal_subset_antisymm(a.sum(b), b)
        a.sum(b) = b
    }
}

/// Adding the bundled zero ideal on the left gives the other ideal.
theorem bundled_zero_ideal_sum_left[R: CommRing](i: Ideal[R]) {
    Ideal[R].zero.sum(i) = i
} by {
    bundled_zero_ideal_subset(i)
    bundled_ideal_sum_eq_right_of_subset(Ideal[R].zero, i)
}

/// Adding the bundled zero ideal on the right gives the other ideal.
theorem bundled_zero_ideal_sum_right[R: CommRing](i: Ideal[R]) {
    i.sum(Ideal[R].zero) = i
} by {
    bundled_zero_ideal_subset(i)
    bundled_ideal_sum_eq_left_of_subset(i, Ideal[R].zero)
}

/// Adding the bundled unit ideal on the left gives the unit ideal.
theorem bundled_unit_ideal_sum_left[R: CommRing](i: Ideal[R]) {
    Ideal[R].unit.sum(i) = Ideal[R].unit
} by {
    bundled_ideal_subset_unit(i)
    bundled_ideal_sum_eq_left_of_subset(Ideal[R].unit, i)
}

/// Adding the bundled unit ideal on the right gives the unit ideal.
theorem bundled_unit_ideal_sum_right[R: CommRing](i: Ideal[R]) {
    i.sum(Ideal[R].unit) = Ideal[R].unit
} by {
    bundled_ideal_subset_unit(i)
    bundled_ideal_sum_eq_right_of_subset(i, Ideal[R].unit)
}

/// True if every element of a set belongs to an ideal.
define set_subset_ideal[R: CommRing](a: Set[R], i: Ideal[R]) -> Bool {
    forall(x: R) {
        a.contains(x) implies i.contains(x)
    }
}

/// Set containment in an ideal is containment in its underlying set.
theorem set_subset_ideal_as_set_eq[R: CommRing](a: Set[R], i: Ideal[R]) {
    set_subset_ideal(a, i) = a.subset(i.as_set)
} by {
    if set_subset_ideal(a, i) {
        forall(x: R) {
            if a.contains(x) {
                i.contains(x)
                ideal_as_set_contains_eq(i, x)
                i.as_set.contains(x)
            }
        }
        a.subset(i.as_set)
    }
    if a.subset(i.as_set) {
        forall(x: R) {
            if a.contains(x) {
                a.subset(i.as_set) = forall(y: R) {
                    a.contains(y) implies i.as_set.contains(y)
                }
                ideal_as_set_contains_eq(i, x)
                i.contains(x)
            }
        }
        set_subset_ideal(a, i)
    }
}

/// The underlying set of an ideal is contained in the ideal.
theorem ideal_as_set_subset_self[R: CommRing](i: Ideal[R]) {
    set_subset_ideal(i.as_set, i)
} by {
    forall(x: R) {
        if i.as_set.contains(x) {
            ideal_as_set_contains_eq(i, x)
            i.contains(x)
        }
    }
}

/// Set containment is preserved by enlarging the target ideal.
theorem set_subset_ideal_trans[R: CommRing](a: Set[R], i: Ideal[R], j: Ideal[R]) {
    set_subset_ideal(a, i) and bundled_ideal_subset(i, j) implies set_subset_ideal(a, j)
} by {
    if set_subset_ideal(a, i) and bundled_ideal_subset(i, j) {
        forall(x: R) {
            if a.contains(x) {
                i.contains(x)
                j.contains(x)
            }
        }
    }
}

/// Set containment in an ideal is preserved by shrinking the set.
theorem set_subset_ideal_of_set_subset[R: CommRing](a: Set[R], b: Set[R], i: Ideal[R]) {
    a.subset(b) and set_subset_ideal(b, i) implies set_subset_ideal(a, i)
} by {
    if a.subset(b) and set_subset_ideal(b, i) {
        forall(x: R) {
            if a.contains(x) {
                a.subset(b) = forall(y: R) {
                    a.contains(y) implies b.contains(y)
                }
                b.contains(x)
                i.contains(x)
            }
        }
    }
}

/// True if an element belongs to every ideal that contains a given set.
define ideal_closure_contains[R: CommRing](a: Set[R], x: R) -> Bool {
    forall(i: Ideal[R]) {
        set_subset_ideal(a, i) implies i.contains(x)
    }
}

/// Membership in the ideal closure gives membership in each ideal that contains the set.
theorem ideal_closure_contains_of_set_subset_raw[R: CommRing](a: Set[R], i: Ideal[R], x: R) {
    ideal_closure_contains(a, x) and set_subset_ideal(a, i) implies i.contains(x)
} by {
    if ideal_closure_contains(a, x) and set_subset_ideal(a, i) {
        ideal_closure_contains(a, x) = forall(j: Ideal[R]) {
            set_subset_ideal(a, j) implies j.contains(x)
        }
        i.contains(x)
    }
}

/// The ideal closure of a set contains zero.
theorem ideal_closure_zero_constraint[R: CommRing](a: Set[R]) {
    ideal_zero_constraint(ideal_closure_contains(a))
} by {
    forall(i: Ideal[R]) {
        if set_subset_ideal(a, i) {
            ideal_contains_zero(i)
            i.contains(R.0)
        }
    }
    ideal_closure_contains(a, R.0)
}

/// The ideal closure of a set is closed under addition.
theorem ideal_closure_add_constraint[R: CommRing](a: Set[R]) {
    ideal_add_constraint(ideal_closure_contains(a))
} by {
    forall(x: R, y: R) {
        if ideal_closure_contains(a, x) and ideal_closure_contains(a, y) {
            forall(i: Ideal[R]) {
                if set_subset_ideal(a, i) {
                    ideal_closure_contains_of_set_subset_raw(a, i, x)
                    ideal_closure_contains_of_set_subset_raw(a, i, y)
                    i.contains(y)
                    ideal_contains_add(i, x, y)
                    i.contains(x + y)
                }
            }
            ideal_closure_contains(a, x + y)
        }
    }
}

/// The ideal closure of a set absorbs multiplication.
theorem ideal_closure_absorb_constraint[R: CommRing](a: Set[R]) {
    ideal_absorb_constraint(ideal_closure_contains(a))
} by {
    forall(r: R, x: R) {
        if ideal_closure_contains(a, x) {
            forall(i: Ideal[R]) {
                if set_subset_ideal(a, i) {
                    ideal_closure_contains_of_set_subset_raw(a, i, x)
                    ideal_contains_mul_left(i, r, x)
                    i.contains(r * x)
                }
            }
            ideal_closure_contains(a, r * x)
        }
    }
}

/// The ideal closure of a set is an ideal.
theorem ideal_closure_is_ideal[R: CommRing](a: Set[R]) {
    is_ideal(ideal_closure_contains(a))
} by {
    ideal_closure_zero_constraint(a)
    ideal_closure_add_constraint(a)
    ideal_closure_absorb_constraint(a)
    is_ideal_from_constraints(ideal_closure_contains(a))
}

/// The smallest ideal containing a set.
let ideal_closure[R: CommRing](a: Set[R]) -> result: Ideal[R] satisfy {
    Ideal.new(ideal_closure_contains(a)) = Option.some(result)
} by {
    ideal_closure_is_ideal(a)
}

attributes Ideal[R: CommRing] {
    /// The smallest ideal containing a set.
    let closure: Set[R] -> Ideal[R] = ideal_closure
}

/// Membership in the closure means membership in every ideal containing the set.
theorem ideal_closure_contains_eq[R: CommRing](a: Set[R], x: R) {
    Ideal[R].closure(a).contains(x) = ideal_closure_contains(a, x)
} by {
    Ideal[R].closure(a).contains(x) = ideal_closure_contains(a, x)
}

/// Every generator belongs to the ideal closure.
theorem set_subset_ideal_closure[R: CommRing](a: Set[R]) {
    set_subset_ideal(a, Ideal[R].closure(a))
} by {
    forall(x: R) {
        if a.contains(x) {
            forall(i: Ideal[R]) {
                if set_subset_ideal(a, i) {
                    i.contains(x)
                }
            }
            ideal_closure_contains(a, x)
            ideal_closure_contains_eq(a, x)
            Ideal[R].closure(a).contains(x)
        }
    }
}

/// A generator is a member of the ideal closure.
theorem ideal_closure_contains_of_set_contains[R: CommRing](a: Set[R], x: R) {
    a.contains(x) implies Ideal[R].closure(a).contains(x)
} by {
    if a.contains(x) {
        set_subset_ideal_closure(a)
        Ideal[R].closure(a).contains(x)
    }
}

/// The ideal closure is contained in any ideal containing the set.
theorem ideal_closure_subset_of_set_subset[R: CommRing](a: Set[R], i: Ideal[R]) {
    set_subset_ideal(a, i) implies bundled_ideal_subset(Ideal[R].closure(a), i)
} by {
    if set_subset_ideal(a, i) {
        forall(x: R) {
            if Ideal[R].closure(a).contains(x) {
                ideal_closure_contains_eq(a, x)
                ideal_closure_contains(a, x)
                ideal_closure_contains_of_set_subset_raw(a, i, x)
            }
        }
    }
}

/// The ideal closure is the least ideal containing the set.
theorem ideal_closure_le_iff_set_subset[R: CommRing](a: Set[R], i: Ideal[R]) {
    bundled_ideal_subset(Ideal[R].closure(a), i) = set_subset_ideal(a, i)
} by {
    if bundled_ideal_subset(Ideal[R].closure(a), i) {
        set_subset_ideal_closure(a)
        set_subset_ideal_trans(a, Ideal[R].closure(a), i)
        set_subset_ideal(a, i)
    }
    if set_subset_ideal(a, i) {
        ideal_closure_subset_of_set_subset(a, i)
        bundled_ideal_subset(Ideal[R].closure(a), i)
    }
}

/// The ideal closure and underlying set maps form a set-containment adjunction.
theorem ideal_closure_as_set_galois_connection[R: CommRing](a: Set[R], i: Ideal[R]) {
    Ideal[R].closure(a).as_set.subset(i.as_set) = a.subset(i.as_set)
} by {
    bundled_ideal_subset_as_set_eq(Ideal[R].closure(a), i)
    ideal_closure_le_iff_set_subset(a, i)
    set_subset_ideal_as_set_eq(a, i)
}

/// The closure of the underlying set of an ideal is the ideal itself.
theorem ideal_closure_as_set[R: CommRing](i: Ideal[R]) {
    Ideal[R].closure(i.as_set) = i
} by {
    ideal_as_set_subset_self(i)
    ideal_closure_subset_of_set_subset(i.as_set, i)
    set_subset_ideal_closure(i.as_set)
    forall(x: R) {
        if i.contains(x) {
            Ideal[R].closure(i.as_set).contains(x)
        }
    }
    bundled_ideal_subset_antisymm(Ideal[R].closure(i.as_set), i)
}

/// Ideal closure is monotone with respect to set inclusion.
theorem ideal_closure_mono[R: CommRing](a: Set[R], b: Set[R]) {
    a.subset(b) implies bundled_ideal_subset(Ideal[R].closure(a), Ideal[R].closure(b))
} by {
    if a.subset(b) {
        set_subset_ideal_closure(b)
        set_subset_ideal_of_set_subset(a, b, Ideal[R].closure(b))
        ideal_closure_subset_of_set_subset(a, Ideal[R].closure(b))
        bundled_ideal_subset(Ideal[R].closure(a), Ideal[R].closure(b))
    }
}

/// Equal sets have equal ideal closures.
theorem ideal_closure_eq_of_set_eq[R: CommRing](a: Set[R], b: Set[R]) {
    a = b implies Ideal[R].closure(a) = Ideal[R].closure(b)
} by {
    if a = b {
        Ideal[R].closure(a) = Ideal[R].closure(b)
    }
}

/// Applying ideal closure twice gives the same ideal.
theorem ideal_closure_idempotent[R: CommRing](a: Set[R]) {
    Ideal[R].closure(Ideal[R].closure(a).as_set) = Ideal[R].closure(a)
} by {
    ideal_closure_as_set(Ideal[R].closure(a))
}

/// The set closure induced by ideal generation.
define ideal_set_closure[R: CommRing](a: Set[R]) -> Set[R] {
    Ideal[R].closure(a).as_set
}

/// The ideal set closure is the underlying set of the generated ideal.
theorem ideal_set_closure_at[R: CommRing](a: Set[R]) {
    ideal_set_closure(a) = Ideal[R].closure(a).as_set
}

/// The ideal set closure contains the original set.
theorem ideal_set_closure_extensive[R: CommRing](a: Set[R]) {
    a.subset(ideal_set_closure(a))
} by {
    ideal_set_closure_at(a)
    set_subset_ideal_closure(a)
    set_subset_ideal_as_set_eq(a, Ideal[R].closure(a))
}

/// The ideal set closure preserves inclusion.
theorem ideal_set_closure_mono[R: CommRing](a: Set[R], b: Set[R]) {
    a.subset(b) implies ideal_set_closure(a).subset(ideal_set_closure(b))
} by {
    if a.subset(b) {
        ideal_closure_mono(a, b)
        bundled_ideal_subset_as_set_eq(Ideal[R].closure(a), Ideal[R].closure(b))
        ideal_set_closure_at(a)
        ideal_set_closure_at(b)
        ideal_set_closure(a).subset(ideal_set_closure(b))
    }
}

/// The ideal set closure is unchanged after two applications.
theorem ideal_set_closure_idempotent[R: CommRing](a: Set[R]) {
    ideal_set_closure(ideal_set_closure(a)) = ideal_set_closure(a)
} by {
    ideal_set_closure_at(a)
    ideal_set_closure_at(ideal_set_closure(a))
    ideal_closure_idempotent(a)
}

/// Ideal set closure is monotone as a set map.
theorem ideal_set_closure_is_monotone[R: CommRing] {
    is_subset_monotone_map(ideal_set_closure[R])
} by {
    forall(a: Set[R], b: Set[R]) {
        if a.subset(b) {
            ideal_set_closure_mono(a, b)
            ideal_set_closure(a).subset(ideal_set_closure(b))
        }
    }
}

/// Ideal set closure is extensive as a set map.
theorem ideal_set_closure_is_extensive[R: CommRing] {
    is_set_extensive_map(ideal_set_closure[R])
} by {
    forall(a: Set[R]) {
        ideal_set_closure_extensive(a)
        a.subset(ideal_set_closure(a))
    }
}

/// Ideal set closure is idempotent as a set map.
theorem ideal_set_closure_is_idempotent[R: CommRing] {
    is_set_idempotent_map(ideal_set_closure[R])
} by {
    forall(a: Set[R]) {
        ideal_set_closure_idempotent(a)
        ideal_set_closure(ideal_set_closure(a)) = ideal_set_closure(a)
    }
}

/// Ideal generation induces a closure operator on sets.
theorem ideal_set_closure_operator[R: CommRing] {
    is_set_closure_operator(ideal_set_closure[R])
} by {
    ideal_set_closure_is_monotone[R]
    ideal_set_closure_is_extensive[R]
    ideal_set_closure_is_idempotent[R]
}

/// The closure of the empty set is the zero ideal.
theorem ideal_closure_empty[R: CommRing] {
    Ideal[R].closure(Set[R].empty_set) = Ideal[R].zero
} by {
    ideal_closure_subset_of_set_subset(Set[R].empty_set, Ideal[R].zero)
    bundled_ideal_subset(Ideal[R].closure(Set[R].empty_set), Ideal[R].zero)
    bundled_zero_ideal_subset(Ideal[R].closure(Set[R].empty_set))
    bundled_ideal_subset_antisymm(Ideal[R].closure(Set[R].empty_set), Ideal[R].zero)
}

/// The closure of the universal set is the unit ideal.
theorem ideal_closure_universal[R: CommRing] {
    Ideal[R].closure(Set[R].universal_set) = Ideal[R].unit
} by {
    bundled_ideal_subset_unit(Ideal[R].closure(Set[R].universal_set))
    forall(x: R) {
        if Ideal[R].unit.contains(x) {
            ideal_closure_contains_of_set_contains(Set[R].universal_set, x)
            Ideal[R].closure(Set[R].universal_set).contains(x)
        }
    }
    bundled_ideal_subset(Ideal[R].unit, Ideal[R].closure(Set[R].universal_set))
    bundled_ideal_subset_antisymm(Ideal[R].closure(Set[R].universal_set), Ideal[R].unit)
}

/// The principal ideal generated by an element is the ideal closure of the singleton containing it.
theorem ideal_closure_singleton_eq_principal[R: CommRing](a: R) {
    Ideal[R].closure(Set[R].singleton(a)) = Ideal[R].principal(a)
} by {
    forall(x: R) {
        if Set[R].singleton(a).contains(x) {
            a = x
            bundled_principal_ideal_contains_generator(a)
            Ideal[R].principal(a).contains(x)
        }
    }
    set_subset_ideal(Set[R].singleton(a), Ideal[R].principal(a))
    ideal_closure_subset_of_set_subset(Set[R].singleton(a), Ideal[R].principal(a))

    set_subset_ideal_closure(Set[R].singleton(a))
    ideal_closure_contains_of_set_contains(Set[R].singleton(a), a)
    forall(x: R) {
        if Ideal[R].principal(a).contains(x) {
            bundled_principal_ideal_contains_eq(a, x)
            let r: R satisfy { x = r * a }
            ideal_contains_mul_left(Ideal[R].closure(Set[R].singleton(a)), r, a)
            Ideal[R].closure(Set[R].singleton(a)).contains(r * a)
            Ideal[R].closure(Set[R].singleton(a)).contains(x)
        }
    }
    bundled_ideal_subset_antisymm(Ideal[R].closure(Set[R].singleton(a)), Ideal[R].principal(a))
}

/// True if an ideal is generated by a finite set.
define ideal_is_finitely_generated[R: CommRing](i: Ideal[R]) -> Bool {
    exists(a: Set[R]) {
        a.is_finite and Ideal[R].closure(a) = i
    }
}

/// A finitely generated ideal has a finite generating set.
theorem ideal_is_finitely_generated_witness[R: CommRing](i: Ideal[R]) {
    ideal_is_finitely_generated(i) implies exists(a: Set[R]) {
        a.is_finite and Ideal[R].closure(a) = i
    }
}

/// An ideal equal to the closure of a finite set is finitely generated.
theorem ideal_is_finitely_generated_of_closure_eq[R: CommRing](a: Set[R], i: Ideal[R]) {
    a.is_finite and Ideal[R].closure(a) = i implies ideal_is_finitely_generated(i)
} by {
    if a.is_finite and Ideal[R].closure(a) = i {
        ideal_is_finitely_generated(i)
    }
}

/// Equality preserves finite generation of ideals.
theorem ideal_is_finitely_generated_of_eq[R: CommRing](i: Ideal[R], j: Ideal[R]) {
    ideal_is_finitely_generated(i) and i = j implies ideal_is_finitely_generated(j)
} by {
    if ideal_is_finitely_generated(i) and i = j {
        ideal_is_finitely_generated_witness(i)
        let a: Set[R] satisfy {
            a.is_finite and Ideal[R].closure(a) = i
        }
        ideal_is_finitely_generated_of_closure_eq(a, j)
        ideal_is_finitely_generated(j)
    }
}

/// The closure of a finite set is a finitely generated ideal.
theorem ideal_closure_is_finitely_generated[R: CommRing](a: Set[R]) {
    a.is_finite implies ideal_is_finitely_generated(Ideal[R].closure(a))
} by {
    if a.is_finite {
        ideal_is_finitely_generated_of_closure_eq(a, Ideal[R].closure(a))
    }
}

/// Principal ideals are finitely generated.
theorem principal_ideal_is_finitely_generated[R: CommRing](a: R) {
    ideal_is_finitely_generated(Ideal[R].principal(a))
} by {
    singleton_set_is_finite(a)
    ideal_closure_singleton_eq_principal(a)
}

/// The zero ideal is finitely generated.
theorem zero_ideal_is_finitely_generated[R: CommRing] {
    ideal_is_finitely_generated(Ideal[R].zero)
} by {
    bundled_principal_ideal_zero_eq_zero[R]
    principal_ideal_is_finitely_generated(R.0)
}

/// The unit ideal is finitely generated.
theorem unit_ideal_is_finitely_generated[R: CommRing] {
    ideal_is_finitely_generated(Ideal[R].unit)
} by {
    bundled_principal_ideal_one_eq_unit[R]
    principal_ideal_is_finitely_generated(R.1)
}

/// The membership predicate of the intersection of an indexed family of ideals.
define ideal_indexed_inter_contains[R: CommRing, I](family: I -> Ideal[R], x: R) -> Bool {
    forall(i: I) { family(i).contains(x) }
}

/// The indexed intersection contains zero.
theorem ideal_indexed_inter_zero_constraint[R: CommRing, I](family: I -> Ideal[R]) {
    ideal_zero_constraint(ideal_indexed_inter_contains(family))
} by {
    forall(i: I) {
        ideal_contains_zero(family(i))
        family(i).contains(R.0)
    }
    ideal_indexed_inter_contains(family, R.0)
}

/// The indexed intersection is closed under addition.
theorem ideal_indexed_inter_add_constraint[R: CommRing, I](family: I -> Ideal[R]) {
    ideal_add_constraint(ideal_indexed_inter_contains(family))
} by {
    forall(a: R, b: R) {
        if ideal_indexed_inter_contains(family, a) and ideal_indexed_inter_contains(family, b) {
            forall(i: I) {
                family(i).contains(b)
                ideal_contains_add(family(i), a, b)
                family(i).contains(a + b)
            }
            ideal_indexed_inter_contains(family, a + b)
        }
    }
}

/// The indexed intersection absorbs multiplication by ring elements.
theorem ideal_indexed_inter_absorb_constraint[R: CommRing, I](family: I -> Ideal[R]) {
    ideal_absorb_constraint(ideal_indexed_inter_contains(family))
} by {
    forall(r: R, a: R) {
        if ideal_indexed_inter_contains(family, a) {
            forall(i: I) {
                ideal_contains_mul_left(family(i), r, a)
                family(i).contains(r * a)
            }
            ideal_indexed_inter_contains(family, r * a)
        }
    }
}

/// The intersection of an indexed family of ideals is an ideal.
theorem ideal_indexed_inter_is_ideal[R: CommRing, I](family: I -> Ideal[R]) {
    is_ideal(ideal_indexed_inter_contains(family))
} by {
    ideal_indexed_inter_zero_constraint(family)
    ideal_indexed_inter_add_constraint(family)
    ideal_indexed_inter_absorb_constraint(family)
    is_ideal_from_constraints(ideal_indexed_inter_contains(family))
}

/// The bundled intersection of an indexed family of ideals.
let ideal_indexed_inter[R: CommRing, I](family: I -> Ideal[R]) -> result: Ideal[R] satisfy {
    Ideal.new(ideal_indexed_inter_contains(family)) = Option.some(result)
} by {
    ideal_indexed_inter_is_ideal(family)
}

/// Membership in the indexed intersection means membership in every member of the family.
theorem ideal_indexed_inter_contains_eq[R: CommRing, I](family: I -> Ideal[R], x: R) {
    ideal_indexed_inter(family).contains(x) = forall(i: I) { family(i).contains(x) }
} by {
    ideal_indexed_inter(family).contains(x) = ideal_indexed_inter_contains(family, x)
}

/// Membership in every member of the family gives membership in the indexed intersection.
theorem ideal_indexed_inter_contains_of_forall[R: CommRing, I](family: I -> Ideal[R], x: R) {
    (forall(i: I) { family(i).contains(x) }) implies ideal_indexed_inter(family).contains(x)
} by {
    if forall(i: I) { family(i).contains(x) } {
        ideal_indexed_inter_contains_eq(family, x)
        ideal_indexed_inter(family).contains(x)
    }
}

/// Membership in the indexed intersection gives membership in each member of the family.
theorem ideal_indexed_inter_contains_at[R: CommRing, I](family: I -> Ideal[R], i: I, x: R) {
    ideal_indexed_inter(family).contains(x) implies family(i).contains(x)
} by {
    if ideal_indexed_inter(family).contains(x) {
        ideal_indexed_inter_contains_eq(family, x)
        family(i).contains(x)
    }
}

/// The indexed intersection is contained in each member of the family.
theorem ideal_indexed_inter_subset[R: CommRing, I](family: I -> Ideal[R], i: I) {
    bundled_ideal_subset(ideal_indexed_inter(family), family(i))
} by {
    forall(x: R) {
        if ideal_indexed_inter(family).contains(x) {
            ideal_indexed_inter_contains_at(family, i, x)
            family(i).contains(x)
        }
    }
}

/// Any ideal contained in every member of the family is contained in the indexed intersection.
theorem ideal_subset_indexed_inter_of_forall[R: CommRing, I](
    c: Ideal[R], family: I -> Ideal[R]
) {
    (forall(i: I) { bundled_ideal_subset(c, family(i)) })
        implies bundled_ideal_subset(c, ideal_indexed_inter(family))
} by {
    if forall(i: I) { bundled_ideal_subset(c, family(i)) } {
        forall(x: R) {
            if c.contains(x) {
                forall(i: I) {
                    bundled_ideal_subset(c, family(i))
                    family(i).contains(x)
                }
                ideal_indexed_inter_contains_of_forall(family, x)
                ideal_indexed_inter(family).contains(x)
            }
        }
    }
}

/// Containment in the indexed intersection is containment in every member of the family.
theorem ideal_subset_indexed_inter_iff[R: CommRing, I](
    c: Ideal[R], family: I -> Ideal[R]
) {
    bundled_ideal_subset(c, ideal_indexed_inter(family)) =
        forall(i: I) { bundled_ideal_subset(c, family(i)) }
} by {
    if bundled_ideal_subset(c, ideal_indexed_inter(family)) {
        forall(i: I) {
            ideal_indexed_inter_subset(family, i)
            bundled_ideal_subset_trans(c, ideal_indexed_inter(family), family(i))
            bundled_ideal_subset(c, family(i))
        }
    }
    if forall(i: I) { bundled_ideal_subset(c, family(i)) } {
        ideal_subset_indexed_inter_of_forall(c, family)
    }
}

/// Indexed intersection is monotone with respect to pointwise inclusion.
theorem ideal_indexed_inter_monotone[R: CommRing, I](
    family_a: I -> Ideal[R], family_b: I -> Ideal[R]
) {
    (forall(i: I) { bundled_ideal_subset(family_a(i), family_b(i)) })
        implies bundled_ideal_subset(ideal_indexed_inter(family_a), ideal_indexed_inter(family_b))
} by {
    if forall(i: I) { bundled_ideal_subset(family_a(i), family_b(i)) } {
        forall(x: R) {
            if ideal_indexed_inter(family_a).contains(x) {
                ideal_indexed_inter_contains_eq(family_a, x)
                forall(i: I) {
                    family_a(i).contains(x)
                    bundled_ideal_subset(family_a(i), family_b(i))
                    family_b(i).contains(x)
                }
                ideal_indexed_inter_contains_of_forall(family_b, x)
                ideal_indexed_inter(family_b).contains(x)
            }
        }
    }
}

/// Equal families have equal indexed intersections.
theorem ideal_indexed_inter_eq_of_family_eq[R: CommRing, I](
    family_a: I -> Ideal[R], family_b: I -> Ideal[R]
) {
    family_a = family_b implies ideal_indexed_inter(family_a) = ideal_indexed_inter(family_b)
} by {
    if family_a = family_b {
        ideal_indexed_inter(family_a) = ideal_indexed_inter(family_b)
    }
}

/// The underlying set of the i-th member of an indexed family of ideals.
define ideal_family_as_set[R: CommRing, I](family: I -> Ideal[R], i: I) -> Set[R] {
    family(i).as_set
}

/// The underlying set of the indexed intersection of ideals is the indexed intersection of the underlying sets.
theorem ideal_indexed_inter_as_set[R: CommRing, I](family: I -> Ideal[R]) {
    ideal_indexed_inter(family).as_set = indexed_intersection(ideal_family_as_set(family))
} by {
    forall(x: R) {
        if ideal_indexed_inter(family).as_set.contains(x) {
            ideal_as_set_contains_eq(ideal_indexed_inter(family), x)
            ideal_indexed_inter_contains_eq(family, x)
            forall(i: I) {
                ideal_contains_as_set_eq(family(i), x)
                ideal_family_as_set(family, i).contains(x)
            }
            indexed_intersection_contains_of_forall(ideal_family_as_set(family), x)
            indexed_intersection(ideal_family_as_set(family)).contains(x)
        }
        if indexed_intersection(ideal_family_as_set(family)).contains(x) {
            forall(i: I) {
                indexed_intersection_contains_at(ideal_family_as_set(family), i, x)
                ideal_as_set_contains_eq(family(i), x)
                family(i).contains(x)
            }
            ideal_indexed_inter_contains_of_forall(family, x)
            ideal_contains_as_set_eq(ideal_indexed_inter(family), x)
            ideal_indexed_inter(family).as_set.contains(x)
        }
        ideal_indexed_inter(family).as_set.contains(x) =
            indexed_intersection(ideal_family_as_set(family)).contains(x)
    }
    forall(x: R) {
        ideal_indexed_inter(family).as_set.contains(x) =
            indexed_intersection(ideal_family_as_set(family)).contains(x)
    }
    set_ext(ideal_indexed_inter(family).as_set, indexed_intersection(ideal_family_as_set(family)))
}

/// True if a subset of a commutative ring is a proper ideal: it does not contain the multiplicative identity.
define ideal_proper_constraint[R: CommRing](contains: R -> Bool) -> Bool {
    not contains(R.1)
}

/// True if a subset of a commutative ring satisfies the primality condition:
/// whenever a product lies in the subset, at least one factor lies in the subset.
define ideal_prime_constraint[R: CommRing](contains: R -> Bool) -> Bool {
    forall(a: R, b: R) {
        contains(a * b) implies contains(a) or contains(b)
    }
}

/// True if a subset of a commutative ring is a prime ideal: an ideal that is
/// proper and whose containment is preserved under reverse-multiplication.
define is_prime_ideal[R: CommRing](contains: R -> Bool) -> Bool {
    is_ideal(contains)
    and ideal_proper_constraint(contains)
    and ideal_prime_constraint(contains)
}

/// is_prime_ideal unfolds to the conjunction of is_ideal with the proper and prime sub-constraints.
theorem is_prime_ideal_unfold[R: CommRing](contains: R -> Bool) {
    is_prime_ideal(contains) = (is_ideal(contains) and ideal_proper_constraint(contains) and ideal_prime_constraint(contains))
}

/// is_ideal together with the proper and prime sub-constraints establishes is_prime_ideal.
theorem is_prime_ideal_from_constraints[R: CommRing](contains: R -> Bool) {
    is_ideal(contains) and ideal_proper_constraint(contains) and ideal_prime_constraint(contains)
        implies is_prime_ideal(contains)
} by {
    is_prime_ideal_unfold(contains)
}

/// A prime ideal is an ideal.
theorem is_prime_ideal_is_ideal[R: CommRing](contains: R -> Bool) {
    is_prime_ideal(contains) implies is_ideal(contains)
} by {
    if is_prime_ideal(contains) {
        is_prime_ideal_unfold(contains)
    }
}

/// A prime ideal does not contain the multiplicative identity.
theorem is_prime_ideal_proper[R: CommRing](contains: R -> Bool) {
    is_prime_ideal(contains) implies not contains(R.1)
} by {
    if is_prime_ideal(contains) {
        is_prime_ideal(contains) = (is_ideal(contains) and ideal_proper_constraint(contains) and ideal_prime_constraint(contains))
        not contains(R.1)
    }
}

/// In a prime ideal, if a product lies in the ideal then at least one factor does.
theorem is_prime_ideal_mem[R: CommRing](contains: R -> Bool, a: R, b: R) {
    is_prime_ideal(contains) and contains(a * b) implies contains(a) or contains(b)
} by {
    if is_prime_ideal(contains) and contains(a * b) {
        is_prime_ideal_unfold(contains)
        ideal_prime_constraint(contains) = forall(x: R, y: R) {
            contains(x * y) implies contains(x) or contains(y)
        }
        contains(a * b) implies contains(a) or contains(b)
    }
}

/// The unit ideal is not a prime ideal: it fails the proper constraint because it contains the multiplicative identity.
theorem unit_ideal_not_prime[R: CommRing] {
    not is_prime_ideal(unit_ideal[R])
} by {
    if is_prime_ideal(unit_ideal[R]) {
        is_prime_ideal_proper(unit_ideal[R])
        false
    }
}

/// True if a subset of a commutative ring is a maximal ideal: a proper ideal
/// that is not strictly contained in any other proper ideal.
define is_maximal_ideal[R: CommRing](contains: R -> Bool) -> Bool {
    is_ideal(contains)
    and ideal_proper_constraint(contains)
    and forall(other: R -> Bool) {
        is_ideal(other) and ideal_subset(contains, other) and ideal_proper_constraint(other)
            implies ideal_subset(other, contains)
    }
}

/// A maximal ideal is an ideal.
theorem is_maximal_ideal_is_ideal[R: CommRing](contains: R -> Bool) {
    is_maximal_ideal(contains) implies is_ideal(contains)
} by {
    if is_maximal_ideal(contains) {
        is_maximal_ideal(contains) = (is_ideal(contains)
            and ideal_proper_constraint(contains)
            and forall(other: R -> Bool) {
                is_ideal(other) and ideal_subset(contains, other) and ideal_proper_constraint(other)
                    implies ideal_subset(other, contains)
            })
    }
}

/// A maximal ideal does not contain the multiplicative identity.
theorem is_maximal_ideal_proper[R: CommRing](contains: R -> Bool) {
    is_maximal_ideal(contains) implies not contains(R.1)
} by {
    if is_maximal_ideal(contains) {
        is_maximal_ideal(contains) = (is_ideal(contains)
            and ideal_proper_constraint(contains)
            and forall(other: R -> Bool) {
                is_ideal(other) and ideal_subset(contains, other) and ideal_proper_constraint(other)
                    implies ideal_subset(other, contains)
            })
        not contains(R.1)
    }
}

/// is_maximal_ideal unfolds to the conjunction of is_ideal, the proper sub-constraint,
/// and the maximality condition on proper ideals.
theorem is_maximal_ideal_unfold[R: CommRing](contains: R -> Bool) {
    is_maximal_ideal(contains) = (is_ideal(contains)
        and ideal_proper_constraint(contains)
        and forall(other: R -> Bool) {
            is_ideal(other) and ideal_subset(contains, other) and ideal_proper_constraint(other)
                implies ideal_subset(other, contains)
        })
}

/// is_ideal together with the proper sub-constraint and the maximality condition
/// establishes is_maximal_ideal.
theorem is_maximal_ideal_from_constraints[R: CommRing](contains: R -> Bool) {
    is_ideal(contains)
        and ideal_proper_constraint(contains)
        and forall(other: R -> Bool) {
            is_ideal(other) and ideal_subset(contains, other) and ideal_proper_constraint(other)
                implies ideal_subset(other, contains)
        }
        implies is_maximal_ideal(contains)
} by {
    is_maximal_ideal_unfold(contains)
}

/// The maximality witness: every proper ideal containing a maximal ideal is contained in it.
theorem is_maximal_ideal_witness[R: CommRing](
    contains: R -> Bool, other: R -> Bool
) {
    is_maximal_ideal(contains)
        and is_ideal(other)
        and ideal_subset(contains, other)
        and ideal_proper_constraint(other)
        implies ideal_subset(other, contains)
} by {
    if is_maximal_ideal(contains)
        and is_ideal(other)
        and ideal_subset(contains, other)
        and ideal_proper_constraint(other) {
        is_maximal_ideal_unfold(contains)
        ideal_subset(other, contains)
    }
}

/// The unit ideal is not a maximal ideal: it fails the proper constraint because it contains the multiplicative identity.
theorem unit_ideal_not_maximal[R: CommRing] {
    not is_maximal_ideal(unit_ideal[R])
} by {
    if is_maximal_ideal(unit_ideal[R]) {
        is_maximal_ideal_proper(unit_ideal[R])
        false
    }
}

/// The sum of an ideal with a principal ideal contains the principal generator.
theorem ideal_sum_principal_contains_generator[R: CommRing](contains: R -> Bool, a: R) {
    is_ideal(contains) implies ideal_sum[R](contains, principal_ideal[R](a), a)
} by {
    if is_ideal(contains) {
        is_ideal_zero(contains)
        principal_ideal_contains_generator(a)
        ideal_sum(contains, principal_ideal[R](a), a)
    }
}

/// If a maximal ideal does not contain a, then the sum of the maximal ideal and the
/// principal ideal generated by a contains the multiplicative identity.
theorem maximal_ideal_sum_principal_contains_one[R: CommRing](contains: R -> Bool, a: R) {
    is_maximal_ideal(contains) and not contains(a)
        implies ideal_sum[R](contains, principal_ideal[R](a), R.1)
} by {
    if is_maximal_ideal(contains) and not contains(a) {
        is_maximal_ideal_is_ideal(contains)
        principal_ideal_is_ideal(a)
        ideal_sum_is_ideal(contains, principal_ideal[R](a))
        ideal_sum_subset_left(contains, principal_ideal[R](a))
        ideal_subset(contains, ideal_sum[R](contains, principal_ideal[R](a)))
        ideal_sum_principal_contains_generator(contains, a)
        if ideal_proper_constraint(ideal_sum[R](contains, principal_ideal[R](a))) {
            is_maximal_ideal_witness(contains, ideal_sum[R](contains, principal_ideal[R](a)))
            ideal_subset(ideal_sum[R](contains, principal_ideal[R](a)), contains) =
                forall(x: R) {
                    ideal_sum[R](contains, principal_ideal[R](a), x) implies contains(x)
                }
            false
        }
        ideal_sum[R](contains, principal_ideal[R](a), R.1)
    }
}

/// If a is not in a maximal ideal but a*b is, then b is in the maximal ideal.
theorem maximal_ideal_prime_step[R: CommRing](contains: R -> Bool, a: R, b: R) {
    is_maximal_ideal(contains) and contains(a * b) and not contains(a) implies contains(b)
} by {
    if is_maximal_ideal(contains) and contains(a * b) and not contains(a) {
        is_maximal_ideal_is_ideal(contains)
        maximal_ideal_sum_principal_contains_one(contains, a)
        let (i: R, p: R) satisfy {
            contains(i) and principal_ideal(a, p) and R.1 = i + p
        }
        let r: R satisfy { p = r * a }
        b * i + b * (r * a) = b * i + r * (a * b)
        b = b * i + r * (a * b)
        is_ideal_absorb(contains, b, i)
        is_ideal_absorb(contains, r, a * b)
        contains(r * (a * b))
        is_ideal_add(contains, b * i, r * (a * b))
        contains(b * i + r * (a * b))
        contains(b)
    }
}

/// In a maximal ideal, if a product lies in the ideal then at least one factor does.
theorem maximal_ideal_prime_constraint[R: CommRing](contains: R -> Bool) {
    is_maximal_ideal(contains) implies ideal_prime_constraint(contains)
} by {
    if is_maximal_ideal(contains) {
        forall(a: R, b: R) {
            if contains(a * b) {
                if not contains(a) {
                    maximal_ideal_prime_step(contains, a, b)
                }
                contains(a) or contains(b)
            }
        }
        ideal_prime_constraint(contains)
    }
}

/// Every maximal ideal is a prime ideal.
theorem is_maximal_ideal_implies_prime[R: CommRing](contains: R -> Bool) {
    is_maximal_ideal(contains) implies is_prime_ideal(contains)
} by {
    if is_maximal_ideal(contains) {
        is_maximal_ideal_is_ideal(contains)
        is_maximal_ideal_proper(contains)
        ideal_proper_constraint(contains)
        maximal_ideal_prime_constraint(contains)
        ideal_prime_constraint(contains)
        is_prime_ideal_from_constraints(contains)
        is_prime_ideal(contains)
    }
}

/// In a maximal ideal, if a product lies in the ideal then at least one factor does.
theorem is_maximal_ideal_mem[R: CommRing](contains: R -> Bool, a: R, b: R) {
    is_maximal_ideal(contains) and contains(a * b) implies contains(a) or contains(b)
} by {
    if is_maximal_ideal(contains) and contains(a * b) {
        is_maximal_ideal_implies_prime(contains)
        is_prime_ideal_mem(contains, a, b)
    }
}

/// A prime ideal of a commutative ring, bundled with its membership predicate.
structure PrimeIdeal[R: CommRing] {
    /// True if the given element is a member of this prime ideal.
    contains: R -> Bool
} constraint {
    is_prime_ideal(contains)
}

/// A prime ideal is in particular an ideal.
theorem prime_ideal_is_ideal[R: CommRing](p: PrimeIdeal[R]) {
    is_ideal(p.contains)
} by {
    is_prime_ideal_is_ideal(p.contains)
}

/// Prime ideal extensionality from pointwise equality of membership.
theorem prime_ideal_ext[R: CommRing](a: PrimeIdeal[R], b: PrimeIdeal[R]) {
    (forall(x: R) { a.contains(x) = b.contains(x) }) implies a = b
} by {
    if forall(x: R) { a.contains(x) = b.contains(x) } {
        a.contains = b.contains
    }
}

/// A prime ideal does not contain the multiplicative identity.
theorem prime_ideal_not_contains_one[R: CommRing](p: PrimeIdeal[R]) {
    not p.contains(R.1)
} by {
    is_prime_ideal_proper(p.contains)
}

/// In a prime ideal, if a product lies in the ideal then at least one factor does.
theorem prime_ideal_contains_mul[R: CommRing](p: PrimeIdeal[R], a: R, b: R) {
    p.contains(a * b) implies p.contains(a) or p.contains(b)
} by {
    if p.contains(a * b) {
        is_prime_ideal(p.contains)
        is_prime_ideal_mem(p.contains, a, b)
    }
}

/// The bundled ideal underlying a prime ideal.
let prime_ideal_as_ideal[R: CommRing](p: PrimeIdeal[R]) -> result: Ideal[R] satisfy {
    Ideal.new(p.contains) = Option.some(result)
} by {
    prime_ideal_is_ideal(p)
}

/// A maximal ideal of a commutative ring, bundled with its membership predicate.
structure MaximalIdeal[R: CommRing] {
    /// True if the given element is a member of this maximal ideal.
    contains: R -> Bool
} constraint {
    is_maximal_ideal(contains)
}

/// A maximal ideal is in particular an ideal.
theorem maximal_ideal_is_ideal[R: CommRing](m: MaximalIdeal[R]) {
    is_ideal(m.contains)
} by {
    is_maximal_ideal_is_ideal(m.contains)
}

/// A maximal ideal is in particular a prime ideal.
theorem maximal_ideal_is_prime[R: CommRing](m: MaximalIdeal[R]) {
    is_prime_ideal(m.contains)
} by {
    is_maximal_ideal_implies_prime(m.contains)
}

/// Maximal ideal extensionality from pointwise equality of membership.
theorem maximal_ideal_ext[R: CommRing](a: MaximalIdeal[R], b: MaximalIdeal[R]) {
    (forall(x: R) { a.contains(x) = b.contains(x) }) implies a = b
} by {
    if forall(x: R) { a.contains(x) = b.contains(x) } {
        a.contains = b.contains
    }
}

/// A maximal ideal does not contain the multiplicative identity.
theorem maximal_ideal_not_contains_one[R: CommRing](m: MaximalIdeal[R]) {
    not m.contains(R.1)
} by {
    is_maximal_ideal_proper(m.contains)
}

/// In a maximal ideal, if a product lies in the ideal then at least one factor does.
theorem maximal_ideal_contains_mul[R: CommRing](m: MaximalIdeal[R], a: R, b: R) {
    m.contains(a * b) implies m.contains(a) or m.contains(b)
} by {
    if m.contains(a * b) {
        is_maximal_ideal(m.contains)
        is_maximal_ideal_mem(m.contains, a, b)
    }
}

/// The maximality witness for a bundled maximal ideal: every proper ideal containing it equals it on its containment.
theorem maximal_ideal_witness[R: CommRing](m: MaximalIdeal[R], other: R -> Bool) {
    is_ideal(other) and ideal_subset(m.contains, other) and ideal_proper_constraint(other)
        implies ideal_subset(other, m.contains)
} by {
    if is_ideal(other) and ideal_subset(m.contains, other) and ideal_proper_constraint(other) {
        is_maximal_ideal(m.contains)
        is_maximal_ideal_witness(m.contains, other)
    }
}

/// The bundled ideal underlying a maximal ideal.
let maximal_ideal_as_ideal[R: CommRing](m: MaximalIdeal[R]) -> result: Ideal[R] satisfy {
    Ideal.new(m.contains) = Option.some(result)
} by {
    maximal_ideal_is_ideal(m)
}

/// The bundled prime ideal underlying a maximal ideal.
let maximal_ideal_as_prime_ideal[R: CommRing](m: MaximalIdeal[R]) -> result: PrimeIdeal[R] satisfy {
    PrimeIdeal.new(m.contains) = Option.some(result)
} by {
    maximal_ideal_is_prime(m)
}

attributes PrimeIdeal[R: CommRing] {
    /// Prime ideal extensionality from pointwise equality of membership.
    let ext = prime_ideal_ext[R]

    /// The bundled ideal obtained by forgetting the prime structure.
    let as_ideal: PrimeIdeal[R] -> Ideal[R] = prime_ideal_as_ideal[R]
}

attributes MaximalIdeal[R: CommRing] {
    /// Maximal ideal extensionality from pointwise equality of membership.
    let ext = maximal_ideal_ext[R]

    /// The bundled ideal obtained by forgetting the maximal structure.
    let as_ideal: MaximalIdeal[R] -> Ideal[R] = maximal_ideal_as_ideal[R]

    /// The bundled prime ideal obtained from a maximal ideal.
    let as_prime_ideal: MaximalIdeal[R] -> PrimeIdeal[R] = maximal_ideal_as_prime_ideal[R]
}

/// Membership in the underlying ideal of a prime ideal coincides with membership in the prime ideal.
theorem prime_ideal_as_ideal_contains_eq[R: CommRing](p: PrimeIdeal[R], x: R) {
    p.as_ideal.contains(x) = p.contains(x)
}

/// Membership in the underlying ideal of a maximal ideal coincides with membership in the maximal ideal.
theorem maximal_ideal_as_ideal_contains_eq[R: CommRing](m: MaximalIdeal[R], x: R) {
    m.as_ideal.contains(x) = m.contains(x)
}

/// Membership in the underlying prime ideal of a maximal ideal coincides with membership in the maximal ideal.
theorem maximal_ideal_as_prime_ideal_contains_eq[R: CommRing](m: MaximalIdeal[R], x: R) {
    m.as_prime_ideal.contains(x) = m.contains(x)
}

/// Membership predicate for the inverse image of an ideal under a ring homomorphism.
define ideal_preimage_contains[R: CommRing, S: CommRing](
    f: RingHom[R, S], i: Ideal[S], x: R
) -> Bool {
    i.contains(f.hom(x))
}

/// The inverse image of an ideal contains zero.
theorem ideal_preimage_zero_constraint[R: CommRing, S: CommRing](
    f: RingHom[R, S], i: Ideal[S]
) {
    ideal_zero_constraint(ideal_preimage_contains(f, i))
} by {
    ring_hom_zero(f)
    ideal_contains_zero(i)
}

/// The inverse image of an ideal is closed under addition.
theorem ideal_preimage_add_constraint[R: CommRing, S: CommRing](
    f: RingHom[R, S], i: Ideal[S]
) {
    ideal_add_constraint(ideal_preimage_contains(f, i))
} by {
    forall(a: R, b: R) {
        if ideal_preimage_contains(f, i, a) and ideal_preimage_contains(f, i, b) {
            i.contains(f.hom(b))
            ideal_contains_add(i, f.hom(a), f.hom(b))
            i.contains(f.hom(a) + f.hom(b))
            ring_hom_add(f, a, b)
            ideal_preimage_contains(f, i, a + b)
        }
    }
}

/// The inverse image of an ideal absorbs multiplication by source-ring elements.
theorem ideal_preimage_absorb_constraint[R: CommRing, S: CommRing](
    f: RingHom[R, S], i: Ideal[S]
) {
    ideal_absorb_constraint(ideal_preimage_contains(f, i))
} by {
    forall(r: R, a: R) {
        if ideal_preimage_contains(f, i, a) {
            ideal_contains_mul_left(i, f.hom(r), f.hom(a))
            i.contains(f.hom(r) * f.hom(a))
            ring_hom_mul(f, r, a)
            ideal_preimage_contains(f, i, r * a)
        }
    }
}

/// The inverse image of an ideal under a ring homomorphism is an ideal.
theorem ideal_preimage_is_ideal[R: CommRing, S: CommRing](
    f: RingHom[R, S], i: Ideal[S]
) {
    is_ideal(ideal_preimage_contains(f, i))
} by {
    ideal_preimage_zero_constraint(f, i)
    ideal_preimage_add_constraint(f, i)
    ideal_preimage_absorb_constraint(f, i)
    is_ideal_from_constraints(ideal_preimage_contains(f, i))
}

/// The inverse image of an ideal under a ring homomorphism.
let ideal_preimage[R: CommRing, S: CommRing](f: RingHom[R, S], i: Ideal[S]) -> result: Ideal[R] satisfy {
    Ideal.new(ideal_preimage_contains(f, i)) = Option.some(result)
} by {
    ideal_preimage_is_ideal(f, i)
}

/// Membership in an inverse-image ideal is membership of the mapped element.
theorem ideal_preimage_contains_eq[R: CommRing, S: CommRing](
    f: RingHom[R, S], i: Ideal[S], x: R
) {
    ideal_preimage(f, i).contains(x) = i.contains(f.hom(x))
} by {
    ideal_preimage(f, i).contains(x) = ideal_preimage_contains(f, i, x)
}

/// An element whose image lies in the target ideal belongs to the inverse-image ideal.
theorem ideal_preimage_contains_of_contains_map[R: CommRing, S: CommRing](
    f: RingHom[R, S], i: Ideal[S], x: R
) {
    i.contains(f.hom(x)) implies ideal_preimage(f, i).contains(x)
} by {
    if i.contains(f.hom(x)) {
        ideal_preimage_contains_eq(f, i, x)
        ideal_preimage(f, i).contains(x)
    }
}

/// A member of the inverse-image ideal maps into the target ideal.
theorem ideal_contains_map_of_preimage_contains[R: CommRing, S: CommRing](
    f: RingHom[R, S], i: Ideal[S], x: R
) {
    ideal_preimage(f, i).contains(x) implies i.contains(f.hom(x))
} by {
    if ideal_preimage(f, i).contains(x) {
        ideal_preimage_contains_eq(f, i, x)
        i.contains(f.hom(x))
    }
}

/// Inverse-image ideals are monotone with respect to target-ideal containment.
theorem ideal_preimage_subset_of_subset[R: CommRing, S: CommRing](
    f: RingHom[R, S], i: Ideal[S], j: Ideal[S]
) {
    bundled_ideal_subset(i, j) implies
        bundled_ideal_subset(ideal_preimage(f, i), ideal_preimage(f, j))
} by {
    if bundled_ideal_subset(i, j) {
        forall(x: R) {
            if ideal_preimage(f, i).contains(x) {
                ideal_contains_map_of_preimage_contains(f, i, x)
                bundled_ideal_subset(i, j) = forall(y: S) {
                    i.contains(y) implies j.contains(y)
                }
                ideal_preimage_contains_of_contains_map(f, j, x)
                ideal_preimage(f, j).contains(x)
            }
        }
    }
}

/// The inverse image of the unit ideal is the unit ideal.
theorem ideal_preimage_unit_eq_unit[R: CommRing, S: CommRing](f: RingHom[R, S]) {
    ideal_preimage(f, Ideal[S].unit) = Ideal[R].unit
} by {
    forall(x: R) {
        ideal_preimage_contains_eq(f, Ideal[S].unit, x)
        bundled_unit_ideal_contains_eq(f.hom(x))
        bundled_unit_ideal_contains_eq(x)
        ideal_preimage(f, Ideal[S].unit).contains(x) = Ideal[R].unit.contains(x)
    }
    ideal_ext(ideal_preimage(f, Ideal[S].unit), Ideal[R].unit)
}

/// The inverse image of an intersection is the intersection of inverse images.
theorem ideal_preimage_intersection[R: CommRing, S: CommRing](
    f: RingHom[R, S], i: Ideal[S], j: Ideal[S]
) {
    ideal_preimage(f, i.intersection(j)) = ideal_preimage(f, i).intersection(ideal_preimage(f, j))
} by {
    forall(x: R) {
        ideal_preimage_contains_eq(f, i.intersection(j), x)
        ideal_preimage_contains_eq(f, i, x)
        ideal_preimage_contains_eq(f, j, x)
        bundled_ideal_inter_contains_eq(i, j, f.hom(x))
        bundled_ideal_inter_contains_eq(ideal_preimage(f, i), ideal_preimage(f, j), x)
        ideal_preimage(f, i.intersection(j)).contains(x) =
            ideal_preimage(f, i).intersection(ideal_preimage(f, j)).contains(x)
    }
    ideal_ext(ideal_preimage(f, i.intersection(j)),
        ideal_preimage(f, i).intersection(ideal_preimage(f, j)))
}

/// The kernel ideal of a ring homomorphism, as the inverse image of the zero ideal.
define ring_hom_kernel[R: CommRing, S: CommRing](f: RingHom[R, S]) -> Ideal[R] {
    ideal_preimage(f, Ideal[S].zero)
}

/// The inverse image of the zero ideal is the kernel ideal.
theorem ideal_preimage_zero_eq_kernel[R: CommRing, S: CommRing](f: RingHom[R, S]) {
    ideal_preimage(f, Ideal[S].zero) = ring_hom_kernel(f)
}

/// Membership in the kernel ideal means the homomorphism maps the element to zero.
theorem ring_hom_kernel_contains_eq[R: CommRing, S: CommRing](
    f: RingHom[R, S], x: R
) {
    ring_hom_kernel(f).contains(x) = (f.hom(x) = S.0)
} by {
    ideal_preimage_contains_eq(f, Ideal[S].zero, x)
    bundled_zero_ideal_contains_eq(f.hom(x))
}

/// An element belongs to the kernel ideal when its image is zero.
theorem ring_hom_kernel_contains_of_maps_to_zero[R: CommRing, S: CommRing](
    f: RingHom[R, S], x: R
) {
    f.hom(x) = S.0 implies ring_hom_kernel(f).contains(x)
} by {
    if f.hom(x) = S.0 {
        ring_hom_kernel_contains_eq(f, x)
        ring_hom_kernel(f).contains(x)
    }
}

/// A kernel-ideal element maps to zero.
theorem ring_hom_maps_to_zero_of_kernel_contains[R: CommRing, S: CommRing](
    f: RingHom[R, S], x: R
) {
    ring_hom_kernel(f).contains(x) implies f.hom(x) = S.0
} by {
    if ring_hom_kernel(f).contains(x) {
        ring_hom_kernel_contains_eq(f, x)
        f.hom(x) = S.0
    }
}

/// The kernel ideal is zero exactly when only zero maps to zero.
theorem ring_hom_kernel_eq_zero_iff_maps_to_zero_eq_zero[R: CommRing, S: CommRing](
    f: RingHom[R, S]
) {
    ring_hom_kernel(f) = Ideal[R].zero = forall(x: R) {
        f.hom(x) = S.0 implies x = R.0
    }
} by {
    if ring_hom_kernel(f) = Ideal[R].zero {
        forall(x: R) {
            if f.hom(x) = S.0 {
                ring_hom_kernel_contains_of_maps_to_zero(f, x)
                bundled_zero_ideal_only_has_zero(x)
                x = R.0
            }
        }
    }
    if forall(x: R) { f.hom(x) = S.0 implies x = R.0 } {
        forall(x: R) {
            if ring_hom_kernel(f).contains(x) {
                ring_hom_maps_to_zero_of_kernel_contains(f, x)
                bundled_zero_ideal_contains_eq(x)
                Ideal[R].zero.contains(x)
            }
        }
        bundled_ideal_subset(ring_hom_kernel(f), Ideal[R].zero)
        bundled_zero_ideal_subset(ring_hom_kernel(f))
        bundled_ideal_subset_antisymm(ring_hom_kernel(f), Ideal[R].zero)
        ring_hom_kernel(f) = Ideal[R].zero
    }
}

/// The inverse image of a prime ideal under a ring homomorphism is prime.
theorem ideal_preimage_prime_is_prime[R: CommRing, S: CommRing](
    f: RingHom[R, S], p: PrimeIdeal[S]
) {
    is_prime_ideal(ideal_preimage_contains(f, p.as_ideal))
} by {
    prime_ideal_is_ideal(p)
    prime_ideal_as_ideal_contains_eq(p, S.1)
    ring_hom_one(f)

    ideal_preimage_is_ideal(f, p.as_ideal)

    if ideal_preimage_contains(f, p.as_ideal, R.1) {
        prime_ideal_not_contains_one(p)
        false
    }

    forall(a: R, b: R) {
        if ideal_preimage_contains(f, p.as_ideal, a * b) {
            ring_hom_mul(f, a, b)
            prime_ideal_contains_mul(p, f.hom(a), f.hom(b))
            if p.contains(f.hom(a)) {
                prime_ideal_as_ideal_contains_eq(p, f.hom(a))
                ideal_preimage_contains(f, p.as_ideal, a) or ideal_preimage_contains(f, p.as_ideal, b)
            }
            if p.contains(f.hom(b)) {
                prime_ideal_as_ideal_contains_eq(p, f.hom(b))
                ideal_preimage_contains(f, p.as_ideal, a) or ideal_preimage_contains(f, p.as_ideal, b)
            }
            ideal_preimage_contains(f, p.as_ideal, a) or ideal_preimage_contains(f, p.as_ideal, b)
        }
    }
    ideal_prime_constraint(ideal_preimage_contains(f, p.as_ideal))

    is_ideal(ideal_preimage_contains(f, p.as_ideal)) and
        ideal_proper_constraint(ideal_preimage_contains(f, p.as_ideal)) and
        ideal_prime_constraint(ideal_preimage_contains(f, p.as_ideal))
    is_prime_ideal_from_constraints(ideal_preimage_contains(f, p.as_ideal))
}

/// The inverse image of a prime ideal under a ring homomorphism.
let prime_ideal_preimage[R: CommRing, S: CommRing](f: RingHom[R, S], p: PrimeIdeal[S]) -> result: PrimeIdeal[R] satisfy {
    PrimeIdeal.new(ideal_preimage_contains(f, p.as_ideal)) = Option.some(result)
} by {
    ideal_preimage_prime_is_prime(f, p)
}

/// Membership in an inverse-image prime ideal is membership of the mapped element.
theorem prime_ideal_preimage_contains_eq[R: CommRing, S: CommRing](
    f: RingHom[R, S], p: PrimeIdeal[S], x: R
) {
    prime_ideal_preimage(f, p).contains(x) = p.contains(f.hom(x))
} by {
    prime_ideal_preimage(f, p).contains(x) = ideal_preimage_contains(f, p.as_ideal, x)
    prime_ideal_as_ideal_contains_eq(p, f.hom(x))
}
