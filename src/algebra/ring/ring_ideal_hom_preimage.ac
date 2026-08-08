/// Functoriality of inverse images of ideals under bundled ring homomorphisms.

from comm_ring import CommRing
from data.basic.functions import compose, identity_fn
from algebra.ring.ring_hom import RingHom, identity_ring_hom, identity_ring_hom_hom,
    compose_ring_hom, compose_ring_hom_hom
from algebra.ring.ideal import Ideal, ideal_ext, bundled_ideal_subset,
    ideal_preimage, ideal_preimage_contains_eq, ideal_preimage_subset_of_subset,
    ring_hom_kernel, ring_hom_kernel_contains_eq

/// Taking the inverse image of an ideal along the identity homomorphism does nothing.
theorem ideal_preimage_identity[R: CommRing](i: Ideal[R]) {
    ideal_preimage(identity_ring_hom[R], i) = i
} by {
    forall(x: R) {
        ideal_preimage_contains_eq(identity_ring_hom[R], i, x)
        identity_ring_hom_hom[R]
        identity_ring_hom[R].hom = identity_fn[R]
        identity_ring_hom[R].hom(x) = identity_fn[R](x)
        identity_fn[R](x) = x
        ideal_preimage(identity_ring_hom[R], i).contains(x) = i.contains(x)
    }
    ideal_ext(ideal_preimage(identity_ring_hom[R], i), i)
}

/// Inverse images of ideals are functorial with respect to composition of homomorphisms.
theorem ideal_preimage_compose[R: CommRing, S: CommRing, T: CommRing](
    f: RingHom[S, T], g: RingHom[R, S], i: Ideal[T]
) {
    ideal_preimage(compose_ring_hom(f, g), i) = ideal_preimage(g, ideal_preimage(f, i))
} by {
    let h = compose_ring_hom(f, g)
    let j = ideal_preimage(f, i)
    let lhs = ideal_preimage(h, i)
    let rhs = ideal_preimage(g, j)
    forall(x: R) {
        ideal_preimage_contains_eq(h, i, x)
        compose_ring_hom_hom(f, g)
        h.hom = compose(f.hom, g.hom)
        h.hom(x) = compose(f.hom, g.hom)(x)
        compose(f.hom, g.hom)(x) = f.hom(g.hom(x))
        ideal_preimage_contains_eq(f, i, g.hom(x))
        j.contains(g.hom(x)) = i.contains(f.hom(g.hom(x)))
        ideal_preimage_contains_eq(g, j, x)
        rhs.contains(x) = j.contains(g.hom(x))
        lhs.contains(x) = rhs.contains(x)
    }
    ideal_ext(lhs, rhs)
    lhs = rhs
    ideal_preimage(compose_ring_hom(f, g), i) = ideal_preimage(g, ideal_preimage(f, i))
}

/// The kernel of a composite homomorphism is the inverse image of the downstream kernel.
theorem ring_hom_kernel_compose_eq_preimage[R: CommRing, S: CommRing, T: CommRing](
    f: RingHom[S, T], g: RingHom[R, S]
) {
    ring_hom_kernel(compose_ring_hom(f, g)) = ideal_preimage(g, ring_hom_kernel(f))
} by {
    let h = compose_ring_hom(f, g)
    let k = ring_hom_kernel(f)
    let lhs = ring_hom_kernel(h)
    let rhs = ideal_preimage(g, k)
    forall(x: R) {
        ring_hom_kernel_contains_eq(h, x)
        compose_ring_hom_hom(f, g)
        h.hom = compose(f.hom, g.hom)
        h.hom(x) = compose(f.hom, g.hom)(x)
        compose(f.hom, g.hom)(x) = f.hom(g.hom(x))
        ring_hom_kernel_contains_eq(f, g.hom(x))
        k.contains(g.hom(x)) = (f.hom(g.hom(x)) = T.0)
        ideal_preimage_contains_eq(g, k, x)
        rhs.contains(x) = k.contains(g.hom(x))
        lhs.contains(x) = rhs.contains(x)
    }
    ideal_ext(lhs, rhs)
    lhs = rhs
    ring_hom_kernel(compose_ring_hom(f, g)) = ideal_preimage(g, ring_hom_kernel(f))
}

/// Containment in a composite kernel is the same as containment in the inverse-image kernel.
theorem bundled_ideal_subset_kernel_compose_iff_preimage_kernel[R: CommRing, S: CommRing, T: CommRing](
    i: Ideal[R], f: RingHom[S, T], g: RingHom[R, S]
) {
    bundled_ideal_subset(i, ring_hom_kernel(compose_ring_hom(f, g))) =
        bundled_ideal_subset(i, ideal_preimage(g, ring_hom_kernel(f)))
} by {
    ring_hom_kernel_compose_eq_preimage(f, g)
    ring_hom_kernel(compose_ring_hom(f, g)) = ideal_preimage(g, ring_hom_kernel(f))
}

/// If an ideal of the middle ring is contained in the downstream kernel, then its
/// inverse image is contained in the composite kernel.
theorem ideal_preimage_subset_kernel_compose_of_subset_kernel[R: CommRing, S: CommRing, T: CommRing](
    f: RingHom[S, T], g: RingHom[R, S], j: Ideal[S]
) {
    bundled_ideal_subset(j, ring_hom_kernel(f)) implies
        bundled_ideal_subset(ideal_preimage(g, j), ring_hom_kernel(compose_ring_hom(f, g)))
} by {
    if bundled_ideal_subset(j, ring_hom_kernel(f)) {
        ideal_preimage_subset_of_subset(g, j, ring_hom_kernel(f))
        bundled_ideal_subset(ideal_preimage(g, j), ideal_preimage(g, ring_hom_kernel(f)))
        ring_hom_kernel_compose_eq_preimage(f, g)
        ring_hom_kernel(compose_ring_hom(f, g)) = ideal_preimage(g, ring_hom_kernel(f))
        bundled_ideal_subset(ideal_preimage(g, j), ring_hom_kernel(compose_ring_hom(f, g)))
    }
}
