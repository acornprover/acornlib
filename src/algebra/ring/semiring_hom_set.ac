from semiring import Semiring
from algebra.semiring_hom import SemiringHom, compose_semiring_hom, compose_semiring_hom_hom
from data.basic.functions import compose
from data.basic.set import Set, set_image, set_preimage, maps_into_set_image, set_image_contains_witness,
    set_image_subset_iff_subset_preimage, set_image_compose, set_preimage_compose,
    set_image_preimage_closure, set_image_preimage_closure_extensive,
    set_image_preimage_closure_monotone, set_image_preimage_closure_idempotent,
    set_preimage_image_kernel, set_preimage_image_kernel_monotone,
    set_preimage_image_kernel_reductive, set_preimage_image_kernel_idempotent,
    is_subset_monotone_map, is_set_extensive_map, is_set_reductive_map,
    is_set_idempotent_map, is_set_closure_operator, is_set_kernel_operator

/// The image of a set under a semiring homomorphism.
define semiring_hom_image[S: Semiring, T: Semiring](f: SemiringHom[S, T], s: Set[S]) -> Set[T] {
    set_image(s, f.hom)
}

/// The preimage of a set under a semiring homomorphism.
define semiring_hom_preimage[S: Semiring, T: Semiring](f: SemiringHom[S, T], s: Set[T]) -> Set[S] {
    set_preimage(f.hom, s)
}

/// The source closure induced by image followed by preimage along a semiring homomorphism.
define semiring_hom_image_preimage_closure[S: Semiring, T: Semiring](
    f: SemiringHom[S, T],
    s: Set[S]
) -> Set[S] {
    set_image_preimage_closure(f.hom, s)
}

/// The target kernel induced by preimage followed by image along a semiring homomorphism.
define semiring_hom_preimage_image_kernel[S: Semiring, T: Semiring](
    f: SemiringHom[S, T],
    s: Set[T]
) -> Set[T] {
    set_preimage_image_kernel(f.hom, s)
}

/// Membership in the image of a semiring homomorphism has a source witness.
theorem semiring_hom_image_contains_witness[S: Semiring, T: Semiring](
    f: SemiringHom[S, T],
    s: Set[S],
    y: T
) {
    semiring_hom_image(f, s).contains(y) implies exists(x: S) {
        s.contains(x) and y = f.hom(x)
    }
} by {
    semiring_hom_image(f, s) = set_image(s, f.hom)
    set_image_contains_witness(s, f.hom, y)
}

/// A member of a set maps into its image under a semiring homomorphism.
theorem semiring_hom_maps_into_image[S: Semiring, T: Semiring](
    f: SemiringHom[S, T],
    s: Set[S],
    x: S
) {
    s.contains(x) implies semiring_hom_image(f, s).contains(f.hom(x))
} by {
    if s.contains(x) {
        maps_into_set_image(s, f.hom, x)
        semiring_hom_image(f, s) = set_image(s, f.hom)
    }
}

/// Image containment is equivalent to source containment in the preimage.
theorem semiring_hom_image_subset_iff_subset_preimage[S: Semiring, T: Semiring](
    f: SemiringHom[S, T],
    s: Set[S],
    u: Set[T]
) {
    semiring_hom_image(f, s).subset(u) = s.subset(semiring_hom_preimage(f, u))
} by {
    semiring_hom_image(f, s) = set_image(s, f.hom)
    semiring_hom_preimage(f, u) = set_preimage(f.hom, u)
    set_image_subset_iff_subset_preimage(s, u, f.hom)
}

/// Image under a composite semiring homomorphism is iterated image.
theorem semiring_hom_image_compose[S: Semiring, T: Semiring, U: Semiring](
    f: SemiringHom[T, U],
    g: SemiringHom[S, T],
    s: Set[S]
) {
    semiring_hom_image(compose_semiring_hom(f, g), s) =
        semiring_hom_image(f, semiring_hom_image(g, s))
} by {
    compose_semiring_hom_hom(f, g)
    semiring_hom_image(compose_semiring_hom(f, g), s) =
        set_image(s, compose(f.hom, g.hom))
    semiring_hom_image(g, s) = set_image(s, g.hom)
    semiring_hom_image(f, semiring_hom_image(g, s)) =
        set_image(set_image(s, g.hom), f.hom)
    set_image_compose(s, g.hom, f.hom)
}

/// Preimage under a composite semiring homomorphism is iterated preimage.
theorem semiring_hom_preimage_compose[S: Semiring, T: Semiring, U: Semiring](
    f: SemiringHom[T, U],
    g: SemiringHom[S, T],
    s: Set[U]
) {
    semiring_hom_preimage(compose_semiring_hom(f, g), s) =
        semiring_hom_preimage(g, semiring_hom_preimage(f, s))
} by {
    compose_semiring_hom_hom(f, g)
    semiring_hom_preimage(compose_semiring_hom(f, g), s) =
        set_preimage(compose(f.hom, g.hom), s)
    semiring_hom_preimage(f, s) = set_preimage(f.hom, s)
    semiring_hom_preimage(g, semiring_hom_preimage(f, s)) =
        set_preimage(g.hom, set_preimage(f.hom, s))
    set_preimage_compose(f.hom, g.hom, s)
}

/// The source closure induced by a semiring homomorphism contains the original set.
theorem semiring_hom_image_preimage_closure_extensive[S: Semiring, T: Semiring](
    f: SemiringHom[S, T],
    s: Set[S]
) {
    s.subset(semiring_hom_image_preimage_closure(f, s))
} by {
    semiring_hom_image_preimage_closure(f, s) = set_image_preimage_closure(f.hom, s)
    set_image_preimage_closure_extensive(f.hom, s)
}

/// The source closure induced by a semiring homomorphism preserves inclusion.
theorem semiring_hom_image_preimage_closure_monotone[S: Semiring, T: Semiring](
    f: SemiringHom[S, T]
) {
    is_subset_monotone_map(semiring_hom_image_preimage_closure(f))
} by {
    forall(s: Set[S], u: Set[S]) {
        if s.subset(u) {
            semiring_hom_image_preimage_closure(f, s) = set_image_preimage_closure(f.hom, s)
            semiring_hom_image_preimage_closure(f, u) = set_image_preimage_closure(f.hom, u)
            set_image_preimage_closure_monotone(f.hom, s, u)
            semiring_hom_image_preimage_closure(f, s).subset(
                semiring_hom_image_preimage_closure(f, u)
            )
        }
    }
}

/// The source closure induced by a semiring homomorphism is idempotent.
theorem semiring_hom_image_preimage_closure_idempotent[S: Semiring, T: Semiring](
    f: SemiringHom[S, T],
    s: Set[S]
) {
    semiring_hom_image_preimage_closure(f, semiring_hom_image_preimage_closure(f, s)) =
        semiring_hom_image_preimage_closure(f, s)
} by {
    semiring_hom_image_preimage_closure(f, s) = set_image_preimage_closure(f.hom, s)
    semiring_hom_image_preimage_closure(f, semiring_hom_image_preimage_closure(f, s)) =
        set_image_preimage_closure(f.hom, set_image_preimage_closure(f.hom, s))
    set_image_preimage_closure_idempotent(f.hom, s)
}

/// The source closure induced by a semiring homomorphism is a set closure operator.
theorem semiring_hom_image_preimage_closure_operator[S: Semiring, T: Semiring](
    f: SemiringHom[S, T]
) {
    is_set_closure_operator(semiring_hom_image_preimage_closure(f))
} by {
    semiring_hom_image_preimage_closure_monotone(f)
    is_subset_monotone_map(semiring_hom_image_preimage_closure(f))
    forall(s: Set[S]) {
        semiring_hom_image_preimage_closure_extensive(f, s)
        s.subset(semiring_hom_image_preimage_closure(f, s))
    }
    is_set_extensive_map(semiring_hom_image_preimage_closure(f))
    forall(s: Set[S]) {
        semiring_hom_image_preimage_closure_idempotent(f, s)
        semiring_hom_image_preimage_closure(f, semiring_hom_image_preimage_closure(f, s)) =
            semiring_hom_image_preimage_closure(f, s)
    }
    is_set_idempotent_map(semiring_hom_image_preimage_closure(f))
    is_set_closure_operator(semiring_hom_image_preimage_closure(f))
}

/// The target kernel induced by a semiring homomorphism lies in the original set.
theorem semiring_hom_preimage_image_kernel_reductive[S: Semiring, T: Semiring](
    f: SemiringHom[S, T],
    s: Set[T]
) {
    semiring_hom_preimage_image_kernel(f, s).subset(s)
} by {
    semiring_hom_preimage_image_kernel(f, s) = set_preimage_image_kernel(f.hom, s)
    set_preimage_image_kernel_reductive(f.hom, s)
}

/// The target kernel induced by a semiring homomorphism preserves inclusion.
theorem semiring_hom_preimage_image_kernel_monotone[S: Semiring, T: Semiring](
    f: SemiringHom[S, T]
) {
    is_subset_monotone_map(semiring_hom_preimage_image_kernel(f))
} by {
    forall(s: Set[T], u: Set[T]) {
        if s.subset(u) {
            semiring_hom_preimage_image_kernel(f, s) = set_preimage_image_kernel(f.hom, s)
            semiring_hom_preimage_image_kernel(f, u) = set_preimage_image_kernel(f.hom, u)
            set_preimage_image_kernel_monotone(f.hom, s, u)
            semiring_hom_preimage_image_kernel(f, s).subset(
                semiring_hom_preimage_image_kernel(f, u)
            )
        }
    }
}

/// The target kernel induced by a semiring homomorphism is idempotent.
theorem semiring_hom_preimage_image_kernel_idempotent[S: Semiring, T: Semiring](
    f: SemiringHom[S, T],
    s: Set[T]
) {
    semiring_hom_preimage_image_kernel(f, semiring_hom_preimage_image_kernel(f, s)) =
        semiring_hom_preimage_image_kernel(f, s)
} by {
    semiring_hom_preimage_image_kernel(f, s) = set_preimage_image_kernel(f.hom, s)
    semiring_hom_preimage_image_kernel(f, semiring_hom_preimage_image_kernel(f, s)) =
        set_preimage_image_kernel(f.hom, set_preimage_image_kernel(f.hom, s))
    set_preimage_image_kernel_idempotent(f.hom, s)
}

/// The target kernel induced by a semiring homomorphism is a set kernel operator.
theorem semiring_hom_preimage_image_kernel_operator[S: Semiring, T: Semiring](
    f: SemiringHom[S, T]
) {
    is_set_kernel_operator(semiring_hom_preimage_image_kernel(f))
} by {
    semiring_hom_preimage_image_kernel_monotone(f)
    is_subset_monotone_map(semiring_hom_preimage_image_kernel(f))
    forall(s: Set[T]) {
        semiring_hom_preimage_image_kernel_reductive(f, s)
        semiring_hom_preimage_image_kernel(f, s).subset(s)
    }
    is_set_reductive_map(semiring_hom_preimage_image_kernel(f))
    forall(s: Set[T]) {
        semiring_hom_preimage_image_kernel_idempotent(f, s)
        semiring_hom_preimage_image_kernel(f, semiring_hom_preimage_image_kernel(f, s)) =
            semiring_hom_preimage_image_kernel(f, s)
    }
    is_set_idempotent_map(semiring_hom_preimage_image_kernel(f))
    is_set_kernel_operator(semiring_hom_preimage_image_kernel(f))
}
