// Homomorphism deepening: preservation theorems for additive, monoid, group,
// ring, and field homomorphisms — subtraction, natural-number powers and
// casts, negation of one, and the behaviour of injectivity under composition —
// together with the compatibility of field homomorphisms with inversion and
// division.

from algebra.group import Group, GroupHom, group_hom_mul, compose_group_hom, compose_group_hom_hom
from algebra.monoid.monoid import Monoid, MonoidHom, monoid_hom_one, monoid_hom_mul
from algebra.add_group import AddGroup, AddGroupHom, add_group_hom_add, add_group_hom_neg,
    compose_add_group_hom, compose_add_group_hom_hom
from algebra.ring.ring import Ring, mul_neg_one_left
from algebra.ring.ring_hom import RingHom, ring_hom_one, ring_hom_add, ring_hom_mul, ring_hom_neg,
    ring_hom_zero, compose_ring_hom, compose_ring_hom_hom
from algebra.ring.ring_theorems_deep import ring_hom_sub
from algebra.field.field import Field, unique_inverse, mul_inverse_right
from algebra.field.field_hom import FieldHom, field_hom_mul, field_hom_one, field_hom_zero
from algebra.field.field_deep import fdiv
from data.basic.functions import compose, is_injective_fn
from nat import Nat, pow_one, pow_add, pow_zero, alt_induction, from_nat, from_nat_zero
numerals Nat

/// An additive group homomorphism preserves subtraction.
theorem add_group_hom_sub[A: AddGroup, B: AddGroup](f: AddGroupHom[A, B], a: A, b: A) {
    f.hom(a - b) = f.hom(a) - f.hom(b)
} by {
    a - b = a + -b
    add_group_hom_add(f, a, -b)
    f.hom(a - b) = f.hom(a) + f.hom(-b)
    add_group_hom_neg(f, b)
    f.hom(-b) = -f.hom(b)
    f.hom(a) - f.hom(b) = f.hom(a) + -f.hom(b)
    f.hom(a - b) = f.hom(a) - f.hom(b)
}

/// A monoid homomorphism preserves natural powers.
theorem monoid_hom_pow[M: Monoid, N: Monoid](f: MonoidHom[M, N], a: M, n: Nat) {
    f.hom(a.pow(n)) = f.hom(a).pow(n)
} by {
    define p(k: Nat) -> Bool {
        f.hom(a.pow(k)) = f.hom(a).pow(k)
    }
    pow_zero(a)
    a.pow(Nat.0) = M.1
    monoid_hom_one(f)
    f.hom(M.1) = N.1
    pow_zero(f.hom(a))
    f.hom(a).pow(Nat.0) = N.1
    f.hom(a.pow(Nat.0)) = f.hom(a).pow(Nat.0)
    p(Nat.0)
    forall(k: Nat) {
        if p(k) {
            pow_add(a, k, Nat.1)
            a.pow(k) * a.pow(Nat.1) = a.pow(k + Nat.1)
            pow_one(a)
            a.pow(Nat.1) = a
            k + Nat.1 = k.suc
            a.pow(k + Nat.1) = a.pow(k.suc)
            a.pow(k) * a = a.pow(k.suc)
            monoid_hom_mul(f, a.pow(k), a)
            f.hom(a.pow(k) * a) = f.hom(a.pow(k)) * f.hom(a)
            p(k) = (f.hom(a.pow(k)) = f.hom(a).pow(k))
            f.hom(a.pow(k)) = f.hom(a).pow(k)
            pow_add(f.hom(a), k, Nat.1)
            f.hom(a).pow(k) * f.hom(a).pow(Nat.1) = f.hom(a).pow(k + Nat.1)
            pow_one(f.hom(a))
            f.hom(a).pow(Nat.1) = f.hom(a)
            f.hom(a).pow(k) * f.hom(a) = f.hom(a).pow(k.suc)
            f.hom(a.pow(k.suc)) = f.hom(a).pow(k.suc)
            p(k.suc)
        }
    }
    p(Nat.0) and forall(k: Nat) { p(k) implies p(k.suc) }
    alt_induction(p)
    forall(k: Nat) { p(k) }
    p(n)
}

/// A ring homomorphism preserves the natural-number embedding.
theorem ring_hom_nat_cast[R: Ring, S: Ring](f: RingHom[R, S], n: Nat) {
    f.hom(from_nat[R](n)) = from_nat[S](n)
} by {
    define p(m: Nat) -> Bool {
        f.hom(from_nat[R](m)) = from_nat[S](m)
    }
    from_nat_zero[R]
    from_nat[R](Nat.0) = R.0
    ring_hom_zero(f)
    f.hom(R.0) = S.0
    from_nat_zero[S]
    from_nat[S](Nat.0) = S.0
    f.hom(from_nat[R](Nat.0)) = from_nat[S](Nat.0)
    p(Nat.0)
    forall(m: Nat) {
        if p(m) {
            from_nat[R](m.suc) = from_nat[R](m) + R.1
            f.hom(from_nat[R](m.suc)) = f.hom(from_nat[R](m) + R.1)
            ring_hom_add(f, from_nat[R](m), R.1)
            f.hom(from_nat[R](m) + R.1) = f.hom(from_nat[R](m)) + f.hom(R.1)
            p(m) = (f.hom(from_nat[R](m)) = from_nat[S](m))
            f.hom(from_nat[R](m)) = from_nat[S](m)
            ring_hom_one(f)
            f.hom(R.1) = S.1
            from_nat[S](m.suc) = from_nat[S](m) + S.1
            f.hom(from_nat[R](m.suc)) = from_nat[S](m.suc)
            p(m.suc)
        }
    }
    p(Nat.0) and forall(m: Nat) { p(m) implies p(m.suc) }
    alt_induction(p)
    forall(m: Nat) { p(m) }
    p(n)
}

/// A ring homomorphism sends negative one to negative one.
theorem ring_hom_neg_one[R: Ring, S: Ring](f: RingHom[R, S]) {
    f.hom(-R.1) = -S.1
} by {
    ring_hom_neg(f, R.1)
    f.hom(-R.1) = -f.hom(R.1)
    ring_hom_one(f)
    f.hom(R.1) = S.1
    -f.hom(R.1) = -S.1
}

/// A ring homomorphism preserves products with a difference on the left.
theorem ring_hom_mul_sub[R: Ring, S: Ring](f: RingHom[R, S], a: R, b: R, c: R) {
    f.hom((a - b) * c) = (f.hom(a) - f.hom(b)) * f.hom(c)
} by {
    ring_hom_mul(f, a - b, c)
    f.hom((a - b) * c) = f.hom(a - b) * f.hom(c)
    ring_hom_sub(f, a, b)
    f.hom(a - b) = f.hom(a) - f.hom(b)
    f.hom(a - b) * f.hom(c) = (f.hom(a) - f.hom(b)) * f.hom(c)
    f.hom((a - b) * c) = (f.hom(a) - f.hom(b)) * f.hom(c)
}

/// A ring homomorphism preserves products with a negated factor.
theorem ring_hom_neg_mul_left[R: Ring, S: Ring](f: RingHom[R, S], a: R, b: R) {
    f.hom(-a * b) = -f.hom(a) * f.hom(b)
} by {
    ring_hom_mul(f, -a, b)
    f.hom(-a * b) = f.hom(-a) * f.hom(b)
    ring_hom_neg(f, a)
    f.hom(-a) = -f.hom(a)
    f.hom(-a) * f.hom(b) = -f.hom(a) * f.hom(b)
    f.hom(-a * b) = -f.hom(a) * f.hom(b)
}

/// A ring homomorphism sends the negation of an element to the negation of its image.
theorem ring_hom_neg_one_mul[R: Ring, S: Ring](f: RingHom[R, S], a: R) {
    f.hom(-R.1 * a) = -f.hom(a)
} by {
    ring_hom_mul(f, -R.1, a)
    f.hom(-R.1 * a) = f.hom(-R.1) * f.hom(a)
    ring_hom_neg_one(f)
    f.hom(-R.1) = -S.1
    f.hom(-R.1) * f.hom(a) = -S.1 * f.hom(a)
    mul_neg_one_left[S](f.hom(a))
    -S.1 * f.hom(a) = -f.hom(a)
    f.hom(-R.1 * a) = -f.hom(a)
}

/// A group homomorphism preserves commuting pairs.
theorem group_hom_commute[G: Group, H: Group](f: GroupHom[G, H], a: G, b: G) {
    a * b = b * a implies f.hom(a) * f.hom(b) = f.hom(b) * f.hom(a)
} by {
    if a * b = b * a {
        group_hom_mul(f, a, b)
        f.hom(a * b) = f.hom(a) * f.hom(b)
        group_hom_mul(f, b, a)
        f.hom(b * a) = f.hom(b) * f.hom(a)
        a * b = b * a
        f.hom(a * b) = f.hom(b * a)
        f.hom(a) * f.hom(b) = f.hom(b) * f.hom(a)
    }
}

/// The composition of injective group homomorphisms is injective.
theorem group_hom_comp_injective[G: Group, H: Group, K: Group](f: GroupHom[H, K], g: GroupHom[G, H]) {
    is_injective_fn(f.hom) and is_injective_fn(g.hom) implies is_injective_fn(compose_group_hom(f, g).hom)
} by {
    if is_injective_fn(f.hom) and is_injective_fn(g.hom) {
        compose_group_hom_hom(f, g)
        compose_group_hom(f, g).hom = compose(f.hom, g.hom)
        is_injective_fn(f.hom) = forall(x: H, y: H) { f.hom(x) = f.hom(y) implies x = y }
        is_injective_fn(g.hom) = forall(x: G, y: G) { g.hom(x) = g.hom(y) implies x = y }
        forall(x: G, y: G) {
            if compose(f.hom, g.hom)(x) = compose(f.hom, g.hom)(y) {
                compose(f.hom, g.hom)(x) = f.hom(g.hom(x))
                compose(f.hom, g.hom)(y) = f.hom(g.hom(y))
                f.hom(g.hom(x)) = f.hom(g.hom(y))
                g.hom(x) = g.hom(y)
                x = y
            }
        }
        is_injective_fn(compose(f.hom, g.hom))
        is_injective_fn(compose_group_hom(f, g).hom)
    }
}

/// The composition of injective additive group homomorphisms is injective.
theorem add_group_hom_comp_injective[A: AddGroup, B: AddGroup, C: AddGroup](
    f: AddGroupHom[B, C],
    g: AddGroupHom[A, B]
) {
    is_injective_fn(f.hom) and is_injective_fn(g.hom) implies is_injective_fn(compose_add_group_hom(f, g).hom)
} by {
    if is_injective_fn(f.hom) and is_injective_fn(g.hom) {
        compose_add_group_hom_hom(f, g)
        compose_add_group_hom(f, g).hom = compose(f.hom, g.hom)
        is_injective_fn(f.hom) = forall(x: B, y: B) { f.hom(x) = f.hom(y) implies x = y }
        is_injective_fn(g.hom) = forall(x: A, y: A) { g.hom(x) = g.hom(y) implies x = y }
        forall(x: A, y: A) {
            if compose(f.hom, g.hom)(x) = compose(f.hom, g.hom)(y) {
                compose(f.hom, g.hom)(x) = f.hom(g.hom(x))
                compose(f.hom, g.hom)(y) = f.hom(g.hom(y))
                f.hom(g.hom(x)) = f.hom(g.hom(y))
                g.hom(x) = g.hom(y)
                x = y
            }
        }
        is_injective_fn(compose(f.hom, g.hom))
        is_injective_fn(compose_add_group_hom(f, g).hom)
    }
}

/// The composition of injective ring homomorphisms is injective.
theorem ring_hom_comp_injective[R: Ring, S: Ring, T: Ring](f: RingHom[S, T], g: RingHom[R, S]) {
    is_injective_fn(f.hom) and is_injective_fn(g.hom) implies is_injective_fn(compose_ring_hom(f, g).hom)
} by {
    if is_injective_fn(f.hom) and is_injective_fn(g.hom) {
        compose_ring_hom_hom(f, g)
        compose_ring_hom(f, g).hom = compose(f.hom, g.hom)
        is_injective_fn(f.hom) = forall(x: S, y: S) { f.hom(x) = f.hom(y) implies x = y }
        is_injective_fn(g.hom) = forall(x: R, y: R) { g.hom(x) = g.hom(y) implies x = y }
        forall(x: R, y: R) {
            if compose(f.hom, g.hom)(x) = compose(f.hom, g.hom)(y) {
                compose(f.hom, g.hom)(x) = f.hom(g.hom(x))
                compose(f.hom, g.hom)(y) = f.hom(g.hom(y))
                f.hom(g.hom(x)) = f.hom(g.hom(y))
                g.hom(x) = g.hom(y)
                x = y
            }
        }
        is_injective_fn(compose(f.hom, g.hom))
        is_injective_fn(compose_ring_hom(f, g).hom)
    }
}

/// A field homomorphism preserves multiplicative inverses.
theorem field_hom_preserves_inverse[F: Field, E: Field](f: FieldHom[F, E], a: F) {
    a != F.0 implies f.hom(a.inverse) = f.hom(a).inverse
} by {
    if a != F.0 {
        mul_inverse_right(a)
        a * a.inverse = F.1
        field_hom_mul(f, a, a.inverse)
        f.hom(a * a.inverse) = f.hom(a) * f.hom(a.inverse)
        f.hom(a * a.inverse) = f.hom(F.1)
        field_hom_one(f)
        f.hom(F.1) = E.1
        f.hom(a) * f.hom(a.inverse) = E.1
        unique_inverse(f.hom(a), f.hom(a.inverse))
        f.hom(a.inverse) = f.hom(a).inverse
    }
}

/// A field homomorphism preserves division.
theorem field_hom_preserves_fdiv[F: Field, E: Field](f: FieldHom[F, E], a: F, b: F) {
    f.hom(fdiv(a, b)) = fdiv(f.hom(a), f.hom(b))
} by {
    fdiv(a, b) = a * b.inverse
    f.hom(fdiv(a, b)) = f.hom(a * b.inverse)
    field_hom_mul(f, a, b.inverse)
    f.hom(a * b.inverse) = f.hom(a) * f.hom(b.inverse)
    if b = F.0 {
        b.inverse = F.0
        field_hom_zero(f)
        f.hom(F.0) = E.0
        f.hom(b.inverse) = E.0
        f.hom(b) = E.0
        f.hom(b).inverse = E.0.inverse
        E.0.inverse = E.0
        f.hom(b.inverse) = f.hom(b).inverse
    } else {
        field_hom_preserves_inverse(f, b)
        f.hom(b.inverse) = f.hom(b).inverse
    }
    f.hom(a) * f.hom(b.inverse) = f.hom(a) * f.hom(b).inverse
    fdiv(f.hom(a), f.hom(b)) = f.hom(a) * f.hom(b).inverse
    f.hom(fdiv(a, b)) = fdiv(f.hom(a), f.hom(b))
}
