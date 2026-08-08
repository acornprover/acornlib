from nat import Nat
from list import List, list_contains_has_greatest, list_contains_has_least
from list import map
from list import pigeonhole_map_into_list, sum
from algebra.add_comm_monoid import AddCommMonoid
from algebra.add_semigroup import add_fn
from algebra.semigroup import mul_fn
from semiring import Semiring
from data.basic.functions import compose, identity_fn, Inhabited, inverse_fn, is_bijection_fn,
    is_injective_fn, bijection_fn_is_injective
from data.basic.logic import exists_intro
from order import LinearOrder, PartialOrder
from data.basic.set import Set, set_ext, finite_constraint, functional_insert, functional_remove,
    list_set, list_set_is_finite, list_set_cardinality_at_most_length,
    unique_list_set_cardinality_is_length, list_set_filter_extracts_subset,
    list_set_contains_eq, set_has_exact_containing_list, set_cardinality_has_exact_unique_list,
    set_image, set_image_is_finite_of_finite, set_image_contains_inverse_of_bijection,
    set_inverse_contains_imp_image_contains, set_image_cardinality_at_most,
    set_image_cardinality_is_of_injective, insert_cardinality_is_suc_of_not_contains,
    remove_insert_cardinality_is_suc_of_contains,
    remove_cardinality_is_of_not_contains, remove_cardinality_is_pred_of_contains,
    insert_then_remove, intersection_cardinality_is_of_subset_left,
    intersection_cardinality_is_of_subset_right,
    difference_cardinality_is_of_disjoint, difference_cardinality_is_zero_of_subset,
    disjoint_union_is_length, union_cardinality_with_difference,
    intersection_is_disjoint_difference, intersection_union_difference_is_self,
    intersection_difference_cardinality_adds, difference_cardinality_is_sub_intersection,
    intersection_cardinality_is_sub_difference, inclusion_exclusion,
    cardinality_is_well_defined, cardinality_is_smallest_cardinality,
    cardinality_is_implies_is_finite
from data.basic.witness import exists_unique, exists_unique_exists, exists_unique_eq,
    set_witness_predicate, set_witness_contains, set_witness_property,
    set_witness_intro, choose_from_set_or_default, choose_from_set_or_default_spec,
    choose_from_set_or_default_contains, choose_from_set_or_default_property,
    choose_from_set_or_default_eq_default

numerals Nat

/// A finite set is a `Set` bundled with a finiteness proof.
structure FiniteSet[T] {
    /// The underlying set.
    underlying_set: Set[T]
} constraint { 
    underlying_set.is_finite
}

let fs_empty[T](s: Set[T]) -> result: FiniteSet[T] satisfy {
    FiniteSet.new(Set[T].empty_set) = Option.some(result)
}

let fs_from_list[T](items: List[T]) -> result: FiniteSet[T] satisfy {
    FiniteSet.new(list_set(items)) = Option.some(result)
}

let fs_insert[T](s: FiniteSet[T], item: T) -> result: FiniteSet[T] satisfy {
    FiniteSet.new(s.underlying_set.insert(item)) = Option.some(result)
}

let fs_remove[T](s: FiniteSet[T], item: T) -> result: FiniteSet[T] satisfy {
    FiniteSet.new(s.underlying_set.remove(item)) = Option.some(result)
}

let fs_union[T](a: FiniteSet[T], b: FiniteSet[T]) -> result: FiniteSet[T] satisfy {
    FiniteSet.new(a.underlying_set.union(b.underlying_set)) = Option.some(result)
}

let fs_intersection[T](a: FiniteSet[T], b: FiniteSet[T]) -> result: FiniteSet[T] satisfy {
    FiniteSet.new(a.underlying_set.intersection(b.underlying_set)) = Option.some(result)
}

let fs_difference[T](a: FiniteSet[T], b: FiniteSet[T]) -> result: FiniteSet[T] satisfy {
    FiniteSet.new(a.underlying_set.difference(b.underlying_set)) = Option.some(result)
}

let fs_image[T, U](s: FiniteSet[T], f: T -> U) -> result: FiniteSet[U] satisfy {
    FiniteSet.new(set_image(s.underlying_set, f)) = Option.some(result)
}

/// Finite set extensionality reduces equality to equality of the underlying sets.
theorem finite_set_ext[T](a: FiniteSet[T], b: FiniteSet[T]) {
    a.underlying_set = b.underlying_set implies a = b
}

/// Finite sets with the same elements are equal.
theorem finite_set_ext_contains[T](a: FiniteSet[T], b: FiniteSet[T]) {
    (forall(x: T) {
        a.underlying_set.contains(x) = b.underlying_set.contains(x)
    }) implies a = b
}

/// Equal finite sets have equal underlying sets.
theorem finite_set_eq_underlying_set[T](a: FiniteSet[T], b: FiniteSet[T]) {
    a = b implies a.underlying_set = b.underlying_set
}

/// Equal finite sets have equal underlying membership at every element.
theorem finite_set_eq_underlying_contains_at[T](a: FiniteSet[T], b: FiniteSet[T], x: T) {
    a = b implies a.underlying_set.contains(x) = b.underlying_set.contains(x)
}

/// Equality of finite sets transports predicates on finite sets.
theorem finite_set_eq_transport_predicate[T](p: FiniteSet[T] -> Bool, a: FiniteSet[T], b: FiniteSet[T]) {
    a = b and p(a) implies p(b)
}

/// Equality of finite sets transports predicates on finite sets in the reverse direction.
theorem finite_set_eq_transport_predicate_rev[T](p: FiniteSet[T] -> Bool, a: FiniteSet[T], b: FiniteSet[T]) {
    a = b and p(b) implies p(a)
}

/// Equality of underlying sets determines equality of finite sets.
theorem finite_set_eq_of_underlying_set_eq[T](a: FiniteSet[T], b: FiniteSet[T]) {
    a.underlying_set = b.underlying_set implies a = b
}

attributes FiniteSet[T] {
    /// Finite set extensionality from equality of the underlying sets.
    let ext = finite_set_ext[T]

    /// Access the underlying set.
    define as_set(self) -> Set[T] {
        self.underlying_set
    }

    /// Membership predicate.
    define contains(self, x: T) -> Bool {
        self.underlying_set.contains(x)
    }

    /// The empty finite set.
    let empty: FiniteSet[T] = fs_empty(Set[T].empty_set)

    /// The finite set whose elements are the elements of the list.
    let from_list: List[T] -> FiniteSet[T] = fs_from_list

    /// Insert preserves finiteness.
    let insert: (FiniteSet[T], T) -> FiniteSet[T] = fs_insert

    /// Remove preserves finiteness.
    let remove: (FiniteSet[T], T) -> FiniteSet[T] = fs_remove

    /// Subset relation lifted from sets.
    define subset_eq(self, other: FiniteSet[T]) -> Bool {
        self.underlying_set.subset(other.underlying_set)
    }

    /// Superset relation lifted from sets.
    define superset_eq(self, other: FiniteSet[T]) -> Bool {
        self.underlying_set.superset(other.underlying_set)
    }

    /// Union of finite sets.
    let union: (FiniteSet[T], FiniteSet[T]) -> FiniteSet[T] = fs_union

    /// Intersection of finite sets.
    let intersection: (FiniteSet[T], FiniteSet[T]) -> FiniteSet[T] = fs_intersection

    /// Difference of finite sets.
    let difference: (FiniteSet[T], FiniteSet[T]) -> FiniteSet[T] = fs_difference

    /// Image of a finite set under a function.
    define image[U](self, f: T -> U) -> FiniteSet[U] {
        fs_image(self, f)
    }

    /// Disjointness predicate.
    define is_disjoint(self, other: FiniteSet[T]) -> Bool {
        self.underlying_set.is_disjoint(other.underlying_set)
    }

    /// Empty predicate.
    define is_empty(self) -> Bool {
        self.underlying_set.is_empty
    }

    /// Cardinality helper lifted from sets.
    define cardinality_at_most(self, n: Nat) -> Bool {
        self.underlying_set.cardinality_at_most(n)
    }

    /// True if the cardinality equals n.
    define cardinality_is(self, n: Nat) -> Bool {
        self.underlying_set.cardinality_is(n)
    }
}

/// The empty finite set has no elements.
theorem finite_set_empty_contains_eq[T](x: T) {
    FiniteSet.empty[T].contains(x) = false
}

/// Membership in a singleton finite set is equality with its element.
theorem finite_set_singleton_contains_eq[T](a: T, x: T) {
    fs_insert(FiniteSet.empty[T], a).contains(x) = (x = a)
}

/// Membership in a finite-set union is membership in either finite set.
theorem finite_set_union_contains_eq[T](a: FiniteSet[T], b: FiniteSet[T], x: T) {
    fs_union(a, b).contains(x) = (a.contains(x) or b.contains(x))
}

/// Membership in a finite-set intersection is membership in both finite sets.
theorem finite_set_intersection_contains_eq[T](a: FiniteSet[T], b: FiniteSet[T], x: T) {
    fs_intersection(a, b).contains(x) = (a.contains(x) and b.contains(x))
}

/// Membership in a finite-set difference is membership in the left finite set and not the right one.
theorem finite_set_difference_contains_eq[T](a: FiniteSet[T], b: FiniteSet[T], x: T) {
    fs_difference(a, b).contains(x) = (a.contains(x) and not b.contains(x))
}

/// A finite subset carries membership into the finite superset.
theorem finite_set_subset_contains[T](a: FiniteSet[T], b: FiniteSet[T], x: T) {
    a.subset_eq(b) and a.contains(x) implies b.contains(x)
}

/// Every finite set is a subset of itself.
theorem finite_set_subset_refl[T](s: FiniteSet[T]) {
    s.subset_eq(s)
}

/// Finite-set subset is transitive.
theorem finite_set_subset_trans[T](a: FiniteSet[T], b: FiniteSet[T], c: FiniteSet[T]) {
    a.subset_eq(b) and b.subset_eq(c) implies a.subset_eq(c)
}

/// Finite-set subset is antisymmetric.
theorem finite_set_subset_antisymm[T](a: FiniteSet[T], b: FiniteSet[T]) {
    a.subset_eq(b) and b.subset_eq(a) implies a = b
}

/// The empty finite set is contained in every finite set.
theorem finite_set_empty_subset[T](s: FiniteSet[T]) {
    FiniteSet.empty[T].subset_eq(s)
}

/// The left operand is contained in a finite-set union.
theorem finite_set_subset_union_left[T](a: FiniteSet[T], b: FiniteSet[T]) {
    a.subset_eq(fs_union(a, b))
}

/// The right operand is contained in a finite-set union.
theorem finite_set_subset_union_right[T](a: FiniteSet[T], b: FiniteSet[T]) {
    b.subset_eq(fs_union(a, b))
}

/// A finite-set union is contained in a finite set exactly when both parts are contained in it.
theorem finite_set_union_subset_eq[T](a: FiniteSet[T], b: FiniteSet[T], c: FiniteSet[T]) {
    fs_union(a, b).subset_eq(c) = (a.subset_eq(c) and b.subset_eq(c))
}

/// A finite set is contained in an intersection exactly when it is contained in both finite sets.
theorem finite_set_subset_intersection_eq[T](a: FiniteSet[T], b: FiniteSet[T], c: FiniteSet[T]) {
    c.subset_eq(fs_intersection(a, b)) = (c.subset_eq(a) and c.subset_eq(b))
}

/// Finite-set union is commutative.
theorem finite_set_union_comm[T](a: FiniteSet[T], b: FiniteSet[T]) {
    fs_union(a, b) = fs_union(b, a)
}

/// Finite-set intersection is commutative.
theorem finite_set_intersection_comm[T](a: FiniteSet[T], b: FiniteSet[T]) {
    fs_intersection(a, b) = fs_intersection(b, a)
}

/// Finite-set intersection is associative.
theorem finite_set_intersection_assoc[T](a: FiniteSet[T], b: FiniteSet[T], c: FiniteSet[T]) {
    fs_intersection(fs_intersection(a, b), c) =
        fs_intersection(a, fs_intersection(b, c))
}

/// Intersection distributes over finite-set union.
theorem finite_set_intersection_union_distrib[T](
    a: FiniteSet[T], b: FiniteSet[T], c: FiniteSet[T]
) {
    fs_intersection(fs_union(a, b), c) =
        fs_union(fs_intersection(a, c), fs_intersection(b, c))
}

/// Finite-set union is idempotent.
theorem finite_set_union_idemp[T](s: FiniteSet[T]) {
    fs_union(s, s) = s
}

/// Finite-set intersection is idempotent.
theorem finite_set_intersection_idemp[T](s: FiniteSet[T]) {
    fs_intersection(s, s) = s
}

/// Union with the empty finite set preserves a finite set.
theorem finite_set_union_empty_right[T](s: FiniteSet[T]) {
    fs_union(s, FiniteSet.empty[T]) = s
}

/// Intersection with the empty finite set is the empty finite set.
theorem finite_set_intersection_empty_right[T](s: FiniteSet[T]) {
    fs_intersection(s, FiniteSet.empty[T]) = FiniteSet.empty[T]
}

/// Membership in a finite-set image is witnessed by a preimage in the finite set.
theorem finite_set_image_contains_eq[T, U](s: FiniteSet[T], f: T -> U, y: U) {
    fs_image(s, f).contains(y) = exists(x: T) {
        s.contains(x) and y = f(x)
    }
}

/// Membership in a finite-set image provides a preimage witness in the finite set.
theorem finite_set_image_contains_witness[T, U](s: FiniteSet[T], f: T -> U, y: U) {
    fs_image(s, f).contains(y) implies exists(x: T) {
        s.contains(x) and y = f(x)
    }
}

/// The image of a member of a finite set belongs to the finite-set image.
theorem finite_set_maps_into_image[T, U](s: FiniteSet[T], f: T -> U, x: T) {
    s.contains(x) implies fs_image(s, f).contains(f(x))
}

/// The image of the empty finite set is empty.
theorem finite_set_image_empty[T, U](f: T -> U) {
    fs_image(FiniteSet.empty[T], f) = FiniteSet.empty[U]
}

/// The image of a singleton finite set is the corresponding singleton finite set.
theorem finite_set_image_singleton[T, U](a: T, f: T -> U) {
    fs_image(fs_insert(FiniteSet.empty[T], a), f) = fs_insert(FiniteSet.empty[U], f(a))
}

/// Finite-set image is monotone with respect to finite-set subset.
theorem finite_set_image_monotone[T, U](a: FiniteSet[T], b: FiniteSet[T], f: T -> U) {
    a.subset_eq(b) implies fs_image(a, f).subset_eq(fs_image(b, f))
}

/// Finite-set image distributes over finite-set union.
theorem finite_set_image_union[T, U](a: FiniteSet[T], b: FiniteSet[T], f: T -> U) {
    fs_image(fs_union(a, b), f) = fs_union(fs_image(a, f), fs_image(b, f))
}

/// The finite-set image of a composite map is the iterated finite-set image.
theorem finite_set_image_compose[A, B, C](s: FiniteSet[A], g: A -> B, f: B -> C) {
    fs_image(fs_image(s, g), f) = fs_image(s, compose(f, g))
}

/// The finite-set image under the identity map is the original finite set.
theorem finite_set_image_identity[T](s: FiniteSet[T]) {
    fs_image(s, identity_fn[T]) = s
}

/// The predicate that an element lies in a finite set and satisfies a condition.
define finite_set_witness_predicate[T](s: FiniteSet[T], p: T -> Bool, x: T) -> Bool {
    set_witness_predicate(s.underlying_set, p, x)
}

/// A finite-set witness belongs to the finite set.
theorem finite_set_witness_contains[T](s: FiniteSet[T], p: T -> Bool, x: T) {
    finite_set_witness_predicate(s, p, x) implies s.contains(x)
}

/// A finite-set witness satisfies the predicate.
theorem finite_set_witness_property[T](s: FiniteSet[T], p: T -> Bool, x: T) {
    finite_set_witness_predicate(s, p, x) implies p(x)
}

/// Finite-set membership together with the predicate gives a finite-set witness.
theorem finite_set_witness_intro[T](s: FiniteSet[T], p: T -> Bool, x: T) {
    s.contains(x) and p(x) implies finite_set_witness_predicate(s, p, x)
}

/// A concrete finite-set witness gives existence of a finite-set witness.
theorem finite_set_witness_exists_of_witness[T](s: FiniteSet[T], p: T -> Bool, x: T) {
    finite_set_witness_predicate(s, p, x) implies exists(y: T) {
        finite_set_witness_predicate(s, p, y)
    }
}

/// A chosen element of a finite set satisfying a condition, with a specified default otherwise.
define choose_from_finite_set_or_default[T](s: FiniteSet[T], p: T -> Bool, default: T) -> T {
    choose_from_set_or_default(s.underlying_set, p, default)
}

/// The chosen finite-set element satisfies the witness predicate when such an element exists.
theorem choose_from_finite_set_or_default_spec[T](s: FiniteSet[T], p: T -> Bool, default: T) {
    exists(x: T) {
        finite_set_witness_predicate(s, p, x)
    } implies finite_set_witness_predicate(s, p, choose_from_finite_set_or_default(s, p, default))
}

/// The chosen finite-set element belongs to the finite set when such an element exists.
theorem choose_from_finite_set_or_default_contains[T](s: FiniteSet[T], p: T -> Bool, default: T) {
    exists(x: T) {
        finite_set_witness_predicate(s, p, x)
    } implies s.contains(choose_from_finite_set_or_default(s, p, default))
}

/// The chosen finite-set element satisfies the condition when such an element exists.
theorem choose_from_finite_set_or_default_property[T](s: FiniteSet[T], p: T -> Bool, default: T) {
    exists(x: T) {
        finite_set_witness_predicate(s, p, x)
    } implies p(choose_from_finite_set_or_default(s, p, default))
}

/// The default is chosen when no finite-set element satisfies the condition.
theorem choose_from_finite_set_or_default_eq_default[T](s: FiniteSet[T], p: T -> Bool, default: T) {
    not exists(x: T) {
        finite_set_witness_predicate(s, p, x)
    } implies choose_from_finite_set_or_default(s, p, default) = default
}

/// A unique finite-set witness determines the chosen element.
theorem choose_from_finite_set_or_default_unique_eq[T](s: FiniteSet[T], p: T -> Bool, default: T, x: T) {
    exists_unique(finite_set_witness_predicate(s, p)) and finite_set_witness_predicate(s, p, x)
    implies choose_from_finite_set_or_default(s, p, default) = x
}

/// A finite set made from a list has cardinality at most the length of the list.
theorem finite_set_from_list_cardinality_at_most_length[T](items: List[T]) {
    fs_from_list(items).cardinality_at_most(items.length)
}

/// A finite set made from a unique list has cardinality equal to the length of the list.
theorem finite_set_from_unique_list_cardinality_is_length[T](items: List[T]) {
    items.is_unique implies fs_from_list(items).cardinality_is(items.length)
}

/// A finite subset of a list-backed set is extracted by filtering the list.
theorem finite_set_filter_extracts_subset[T](items: List[T], s: FiniteSet[T]) {
    s.underlying_set.subset(list_set(items)) implies
    fs_from_list(items.filter(s.underlying_set.contains)) = s
}

/// A finite subset of a list-backed set has a list representation.
theorem finite_set_subset_has_list[T](items: List[T], s: FiniteSet[T]) {
    items.contains_set(s.underlying_set) implies exists(subitems: List[T]) {
        fs_from_list(subitems) = s
    }
}

/// A finite set has a list representation with the same elements.
theorem finite_set_has_list[T](s: FiniteSet[T]) {
    exists(items: List[T]) {
        fs_from_list(items) = s
    }
}

/// A finite set has a unique list representation with the same elements.
theorem finite_set_has_unique_list[T](s: FiniteSet[T]) {
    exists(items: List[T]) {
        fs_from_list(items) = s and items.is_unique
    }
}

/// A nonempty finite subset of a linear order has a least element.
theorem finite_set_nonempty_has_least[T: LinearOrder](s: FiniteSet[T]) {
    not s.is_empty implies exists(least: T) {
        s.contains(least) and forall(item: T) {
            s.contains(item) implies least <= item
        }
    }
}

/// A nonempty finite subset of a linear order has a greatest element.
theorem finite_set_nonempty_has_greatest[T: LinearOrder](s: FiniteSet[T]) {
    not s.is_empty implies exists(greatest: T) {
        s.contains(greatest) and forall(item: T) {
            s.contains(item) implies item <= greatest
        }
    }
}

/// A nonempty finite subset of a partial order has a maximal element.
theorem finite_set_nonempty_has_maximal[T: PartialOrder](s: FiniteSet[T]) {
    not s.is_empty implies exists(maximal: T) {
        s.contains(maximal) and forall(item: T) {
            s.contains(item) and maximal <= item implies item = maximal
        }
    }
}

/// A nonempty finite subset of a partial order has a minimal element.
theorem finite_set_nonempty_has_minimal[T: PartialOrder](s: FiniteSet[T]) {
    not s.is_empty implies exists(minimal: T) {
        s.contains(minimal) and forall(item: T) {
            s.contains(item) and item <= minimal implies item = minimal
        }
    }
}

/// Two least elements of a finite subset of a partial order are equal.
theorem finite_set_least_unique[T: PartialOrder](s: FiniteSet[T], a: T, b: T) {
    s.contains(a) and s.contains(b) and
    (forall(item: T) { s.contains(item) implies a <= item }) and
    (forall(item: T) { s.contains(item) implies b <= item })
    implies a = b
}

/// Two greatest elements of a finite subset of a partial order are equal.
theorem finite_set_greatest_unique[T: PartialOrder](s: FiniteSet[T], a: T, b: T) {
    s.contains(a) and s.contains(b) and
    (forall(item: T) { s.contains(item) implies item <= a }) and
    (forall(item: T) { s.contains(item) implies item <= b })
    implies a = b
}

/// Inclusion reverses the comparison of least elements of finite subsets of a partial order.
theorem finite_set_least_lte_of_subset[T: PartialOrder](
    s: FiniteSet[T], t: FiniteSet[T], least_s: T, least_t: T
) {
    s.subset_eq(t) and s.contains(least_s) and
    (forall(item: T) { s.contains(item) implies least_s <= item }) and
    t.contains(least_t) and
    (forall(item: T) { t.contains(item) implies least_t <= item })
    implies least_t <= least_s
}

/// Inclusion preserves the comparison of greatest elements of finite subsets of a partial order.
theorem finite_set_greatest_lte_of_subset[T: PartialOrder](
    s: FiniteSet[T], t: FiniteSet[T], greatest_s: T, greatest_t: T
) {
    s.subset_eq(t) and s.contains(greatest_s) and
    (forall(item: T) { s.contains(item) implies item <= greatest_s }) and
    t.contains(greatest_t) and
    (forall(item: T) { t.contains(item) implies item <= greatest_t })
    implies greatest_s <= greatest_t
}

/// A least element of a finite subset of a partial order lies below every greatest element.
theorem finite_set_least_lte_greatest[T: PartialOrder](
    s: FiniteSet[T], least: T, greatest: T
) {
    s.contains(least) and
    (forall(item: T) { s.contains(item) implies least <= item }) and
    s.contains(greatest) and
    (forall(item: T) { s.contains(item) implies item <= greatest })
    implies least <= greatest
}

/// A nonempty finite subset of a linear order has comparable least and greatest elements.
theorem finite_set_nonempty_has_least_and_greatest[T: LinearOrder](s: FiniteSet[T]) {
    not s.is_empty implies exists(least: T, greatest: T) {
        s.contains(least) and
        (forall(item: T) { s.contains(item) implies least <= item }) and
        s.contains(greatest) and
        (forall(item: T) { s.contains(item) implies item <= greatest }) and
        least <= greatest
    }
}

/// The least element of a finite-set union lies below the least element of its left operand.
theorem finite_set_union_least_lte_left[T: PartialOrder](
    s: FiniteSet[T], t: FiniteSet[T], least_s: T, least_union: T
) {
    s.contains(least_s) and
    (forall(item: T) { s.contains(item) implies least_s <= item }) and
    fs_union(s, t).contains(least_union) and
    (forall(item: T) { fs_union(s, t).contains(item) implies least_union <= item })
    implies least_union <= least_s
}

/// The least element of a finite-set union lies below the least element of its right operand.
theorem finite_set_union_least_lte_right[T: PartialOrder](
    s: FiniteSet[T], t: FiniteSet[T], least_t: T, least_union: T
) {
    t.contains(least_t) and
    (forall(item: T) { t.contains(item) implies least_t <= item }) and
    fs_union(s, t).contains(least_union) and
    (forall(item: T) { fs_union(s, t).contains(item) implies least_union <= item })
    implies least_union <= least_t
}

/// The greatest element of a finite-set union lies above the greatest element of its left operand.
theorem finite_set_union_greatest_gte_left[T: PartialOrder](
    s: FiniteSet[T], t: FiniteSet[T], greatest_s: T, greatest_union: T
) {
    s.contains(greatest_s) and
    (forall(item: T) { s.contains(item) implies item <= greatest_s }) and
    fs_union(s, t).contains(greatest_union) and
    (forall(item: T) { fs_union(s, t).contains(item) implies item <= greatest_union })
    implies greatest_s <= greatest_union
}

/// The greatest element of a finite-set union lies above the greatest element of its right operand.
theorem finite_set_union_greatest_gte_right[T: PartialOrder](
    s: FiniteSet[T], t: FiniteSet[T], greatest_t: T, greatest_union: T
) {
    t.contains(greatest_t) and
    (forall(item: T) { t.contains(item) implies item <= greatest_t }) and
    fs_union(s, t).contains(greatest_union) and
    (forall(item: T) { fs_union(s, t).contains(item) implies item <= greatest_union })
    implies greatest_t <= greatest_union
}

/// The minimum of two least elements belongs to the union of their finite sets.
theorem finite_set_union_contains_min_of_leasts[T: LinearOrder](
    s: FiniteSet[T], t: FiniteSet[T], least_s: T, least_t: T
) {
    s.contains(least_s) and t.contains(least_t)
    implies fs_union(s, t).contains(least_s.min(least_t))
}

/// The minimum of lower bounds for two finite sets is a lower bound for their union.
theorem finite_set_union_min_is_lower_bound[T: LinearOrder](
    s: FiniteSet[T], t: FiniteSet[T], least_s: T, least_t: T
) {
    (forall(item: T) { s.contains(item) implies least_s <= item }) and
    (forall(item: T) { t.contains(item) implies least_t <= item })
    implies forall(item: T) {
        fs_union(s, t).contains(item) implies least_s.min(least_t) <= item
    }
}

/// A least element of a finite-set union is the minimum of the operands' least elements.
theorem finite_set_union_least_eq_min[T: LinearOrder](
    s: FiniteSet[T], t: FiniteSet[T], least_s: T, least_t: T, least_union: T
) {
    s.contains(least_s) and
    (forall(item: T) { s.contains(item) implies least_s <= item }) and
    t.contains(least_t) and
    (forall(item: T) { t.contains(item) implies least_t <= item }) and
    fs_union(s, t).contains(least_union) and
    (forall(item: T) { fs_union(s, t).contains(item) implies least_union <= item })
    implies least_union = least_s.min(least_t)
}

/// The maximum of two greatest elements belongs to the union of their finite sets.
theorem finite_set_union_contains_max_of_greatests[T: LinearOrder](
    s: FiniteSet[T], t: FiniteSet[T], greatest_s: T, greatest_t: T
) {
    s.contains(greatest_s) and t.contains(greatest_t)
    implies fs_union(s, t).contains(greatest_s.max(greatest_t))
}

/// The maximum of upper bounds for two finite sets is an upper bound for their union.
theorem finite_set_union_max_is_upper_bound[T: LinearOrder](
    s: FiniteSet[T], t: FiniteSet[T], greatest_s: T, greatest_t: T
) {
    (forall(item: T) { s.contains(item) implies item <= greatest_s }) and
    (forall(item: T) { t.contains(item) implies item <= greatest_t })
    implies forall(item: T) {
        fs_union(s, t).contains(item) implies item <= greatest_s.max(greatest_t)
    }
}

/// A greatest element of a finite-set union is the maximum of the operands' greatest elements.
theorem finite_set_union_greatest_eq_max[T: LinearOrder](
    s: FiniteSet[T], t: FiniteSet[T], greatest_s: T, greatest_t: T, greatest_union: T
) {
    s.contains(greatest_s) and
    (forall(item: T) { s.contains(item) implies item <= greatest_s }) and
    t.contains(greatest_t) and
    (forall(item: T) { t.contains(item) implies item <= greatest_t }) and
    fs_union(s, t).contains(greatest_union) and
    (forall(item: T) { fs_union(s, t).contains(item) implies item <= greatest_union })
    implies greatest_union = greatest_s.max(greatest_t)
}

/// Every member equals an element which is both least and greatest.
theorem finite_set_member_eq_of_least_and_greatest[T: PartialOrder](
    s: FiniteSet[T], extremum: T, item: T
) {
    s.contains(extremum) and
    (forall(member: T) { s.contains(member) implies extremum <= member }) and
    (forall(member: T) { s.contains(member) implies member <= extremum }) and
    s.contains(item)
    implies item = extremum
}

/// A singleton finite set is contained in every finite set containing its element.
theorem finite_set_singleton_subset_of_contains[T](s: FiniteSet[T], item: T) {
    s.contains(item) implies fs_insert(FiniteSet.empty[T], item).subset_eq(s)
}

/// A finite subset with the same least and greatest element is contained in its singleton.
theorem finite_set_subset_singleton_of_least_and_greatest[T: PartialOrder](
    s: FiniteSet[T], extremum: T
) {
    s.contains(extremum) and
    (forall(item: T) { s.contains(item) implies extremum <= item }) and
    (forall(item: T) { s.contains(item) implies item <= extremum })
    implies s.subset_eq(fs_insert(FiniteSet.empty[T], extremum))
}

/// A finite subset with the same least and greatest element is a singleton.
theorem finite_set_eq_singleton_of_least_and_greatest[T: PartialOrder](
    s: FiniteSet[T], extremum: T
) {
    s.contains(extremum) and
    (forall(item: T) { s.contains(item) implies extremum <= item }) and
    (forall(item: T) { s.contains(item) implies item <= extremum })
    implies s = fs_insert(FiniteSet.empty[T], extremum)
}

/// The element of a singleton finite subset is a lower bound.
theorem finite_set_singleton_element_is_lower_bound[T: PartialOrder](item: T) {
    forall(member: T) {
        fs_insert(FiniteSet.empty[T], item).contains(member) implies item <= member
    }
}

/// The element of a singleton finite subset is an upper bound.
theorem finite_set_singleton_element_is_upper_bound[T: PartialOrder](item: T) {
    forall(member: T) {
        fs_insert(FiniteSet.empty[T], item).contains(member) implies member <= item
    }
}

/// A singleton finite set has cardinality one.
theorem finite_set_singleton_cardinality_is_one[T](item: T) {
    fs_insert(FiniteSet.empty[T], item).cardinality_is(Nat.1)
}

/// A finite set of cardinality one is the singleton of any element it contains.
theorem finite_set_eq_singleton_of_cardinality_one_contains[T](s: FiniteSet[T], item: T) {
    s.cardinality_is(Nat.1) and s.contains(item) implies
        s = fs_insert(FiniteSet.empty[T], item)
}

/// Every element of a cardinality-one finite subset is a lower bound.
theorem finite_set_cardinality_one_element_is_lower_bound[T: PartialOrder](
    s: FiniteSet[T], item: T
) {
    s.cardinality_is(Nat.1) and s.contains(item) implies forall(member: T) {
        s.contains(member) implies item <= member
    }
}

/// Every element of a cardinality-one finite subset is an upper bound.
theorem finite_set_cardinality_one_element_is_upper_bound[T: PartialOrder](
    s: FiniteSet[T], item: T
) {
    s.cardinality_is(Nat.1) and s.contains(item) implies forall(member: T) {
        s.contains(member) implies member <= item
    }
}

/// A finite set containing two distinct elements has cardinality at least two.
theorem finite_set_cardinality_at_least_two_of_contains_ne[T](
    s: FiniteSet[T], a: T, b: T, n: Nat
) {
    s.cardinality_is(n) and s.contains(a) and s.contains(b) and a != b implies Nat.2 <= n
}

/// Distinct least and greatest elements force cardinality at least two.
theorem finite_set_distinct_extrema_cardinality_at_least_two[T: PartialOrder](
    s: FiniteSet[T], least: T, greatest: T, n: Nat
) {
    s.cardinality_is(n) and
    s.contains(least) and
    (forall(item: T) { s.contains(item) implies least <= item }) and
    s.contains(greatest) and
    (forall(item: T) { s.contains(item) implies item <= greatest }) and
    least != greatest
    implies Nat.2 <= n
}

/// A finite set with exact cardinality is represented by a unique list of that length.
theorem finite_set_cardinality_has_exact_unique_list[T](s: FiniteSet[T], n: Nat) {
    s.cardinality_is(n) implies exists(items: List[T]) {
        fs_from_list(items) = s and items.is_unique and items.length = n
    }
}

/// The image of a cardinality-bounded finite set has the same cardinality bound.
theorem finite_set_image_cardinality_at_most[T, U](s: FiniteSet[T], f: T -> U, n: Nat) {
    s.cardinality_at_most(n) implies fs_image(s, f).cardinality_at_most(n)
}

/// The image of a finite set under an injective map has the same exact cardinality.
theorem finite_set_image_cardinality_is_of_injective[T, U](s: FiniteSet[T], f: T -> U, n: Nat) {
    is_injective_fn(f) and s.cardinality_is(n) implies fs_image(s, f).cardinality_is(n)
}

/// The image of a finite set under a bijection has the same exact cardinality.
theorem finite_set_image_cardinality_is_of_bijection[T, U](s: FiniteSet[T], f: T -> U, n: Nat) {
    is_bijection_fn(f) and s.cardinality_is(n) implies fs_image(s, f).cardinality_is(n)
}

/// Membership in a finite-set image gives membership of the chosen inverse.
theorem finite_set_image_contains_inverse_of_bijection[T: Inhabited, U](s: FiniteSet[T], f: T -> U, y: U) {
    is_bijection_fn(f) and fs_image(s, f).contains(y) implies s.contains(inverse_fn(f, y))
}

/// Membership of the chosen inverse gives membership in a finite-set image.
theorem finite_set_inverse_contains_imp_image_contains[T: Inhabited, U](s: FiniteSet[T], f: T -> U, y: U) {
    is_bijection_fn(f) and s.contains(inverse_fn(f, y)) implies fs_image(s, f).contains(y)
}

/// Membership in a finite-set image is membership of the chosen inverse.
theorem finite_set_image_contains_iff_inverse_of_bijection[T: Inhabited, U](s: FiniteSet[T], f: T -> U, y: U) {
    is_bijection_fn(f) implies fs_image(s, f).contains(y) = s.contains(inverse_fn(f, y))
}

/// A map from a finite set into a shorter containing list has a collision.
theorem finite_set_pigeonhole_into_list[T, U](s: FiniteSet[T], targets: List[U], f: T -> U, n: Nat) {
    s.cardinality_is(n) and targets.length < n and
    forall(x: T) {
        s.contains(x) implies targets.contains(f(x))
    } implies exists(x: T, y: T) {
        x != y and f(x) = f(y)
    }
}

/// A map from a finite set into a smaller finite set has a collision.
theorem finite_set_pigeonhole_into_finite_set[T, U](s: FiniteSet[T], t: FiniteSet[U],
    f: T -> U, n_s: Nat, n_t: Nat) {
    s.cardinality_is(n_s) and t.cardinality_is(n_t) and n_t < n_s and
    forall(x: T) {
        s.contains(x) implies t.contains(f(x))
    } implies exists(x: T, y: T) {
        x != y and f(x) = f(y)
    }
}

/// Exact cardinality of a finite set is unique.
theorem finite_set_cardinality_is_well_defined[T](s: FiniteSet[T], n1: Nat, n2: Nat) {
    s.cardinality_is(n1) and s.cardinality_is(n2) implies n1 = n2
}

/// Every finite set has an exact cardinality.
theorem finite_set_cardinality_exists[T](s: FiniteSet[T]) {
    exists(n: Nat) {
        s.cardinality_is(n)
    }
}

/// Exact cardinality gives the corresponding upper cardinality bound.
theorem finite_set_cardinality_is_smallest_cardinality[T](s: FiniteSet[T], n: Nat) {
    s.cardinality_is(n) implies s.cardinality_at_most(n)
}

/// Exact cardinality implies finiteness of the underlying set.
theorem finite_set_cardinality_is_implies_underlying_is_finite[T](s: FiniteSet[T], n: Nat) {
    s.cardinality_is(n) implies s.underlying_set.is_finite
}

/// Removing a newly inserted absent element recovers the original finite set.
theorem finite_set_insert_remove_of_not_contains[T](s: FiniteSet[T], item: T) {
    not s.contains(item) implies s.insert(item).remove(item) = s
}

/// Inserting a new element increases exact cardinality by one.
theorem finite_set_insert_cardinality_is_suc_of_not_contains[T](s: FiniteSet[T], item: T, n: Nat) {
    s.cardinality_is(n) and not s.contains(item) implies fs_insert(s, item).cardinality_is(n + Nat.1)
}

/// Removing an element and adding it back recovers the exact cardinality.
theorem finite_set_remove_insert_cardinality_is_suc_of_contains[T](s: FiniteSet[T], item: T, n: Nat) {
    fs_remove(s, item).cardinality_is(n) and s.contains(item) implies s.cardinality_is(n + Nat.1)
}

/// Removing an absent element preserves exact cardinality.
theorem finite_set_remove_cardinality_is_of_not_contains[T](s: FiniteSet[T], item: T, n: Nat) {
    s.cardinality_is(n) and not s.contains(item) implies fs_remove(s, item).cardinality_is(n)
}

/// Removing a present element decreases exact cardinality by one.
theorem finite_set_remove_cardinality_is_pred_of_contains[T](s: FiniteSet[T], item: T, n: Nat) {
    s.cardinality_is(n + Nat.1) and s.contains(item) implies fs_remove(s, item).cardinality_is(n)
}

/// Intersecting a finite set with one of its supersets preserves exact cardinality.
theorem finite_set_intersection_cardinality_is_of_subset_left[T](s: FiniteSet[T], t: FiniteSet[T], n: Nat) {
    s.subset_eq(t) and s.cardinality_is(n) implies fs_intersection(s, t).cardinality_is(n)
}

/// Intersecting a finite set with one of its subsets has the subset's exact cardinality.
theorem finite_set_intersection_cardinality_is_of_subset_right[T](s: FiniteSet[T], t: FiniteSet[T], n: Nat) {
    t.subset_eq(s) and t.cardinality_is(n) implies fs_intersection(s, t).cardinality_is(n)
}

/// Difference by a disjoint finite set preserves exact cardinality.
theorem finite_set_difference_cardinality_is_of_disjoint[T](s: FiniteSet[T], t: FiniteSet[T], n: Nat) {
    s.cardinality_is(n) and s.is_disjoint(t) implies fs_difference(s, t).cardinality_is(n)
}

/// Difference by a finite superset has cardinality zero.
theorem finite_set_difference_cardinality_is_zero_of_subset[T](s: FiniteSet[T], t: FiniteSet[T]) {
    s.subset_eq(t) implies fs_difference(s, t).cardinality_is(Nat.0)
}

/// Disjoint finite sets have union cardinality equal to the sum of their cardinalities.
theorem finite_set_disjoint_union_cardinality_is[T](a: FiniteSet[T], b: FiniteSet[T], n1: Nat, n2: Nat) {
    a.cardinality_is(n1) and b.cardinality_is(n2) and a.is_disjoint(b)
    implies fs_union(a, b).cardinality_is(n1 + n2)
}

/// A union decomposed into a set and the outside difference has additive cardinality.
theorem finite_set_union_cardinality_with_difference[T](s: FiniteSet[T], t: FiniteSet[T], n_s: Nat, n_diff: Nat) {
    s.cardinality_is(n_s) and fs_difference(t, s).cardinality_is(n_diff)
    implies fs_union(s, t).cardinality_is(n_s + n_diff)
}

/// The intersection of finite sets is disjoint from the left difference.
theorem finite_set_intersection_is_disjoint_difference[T](s: FiniteSet[T], t: FiniteSet[T]) {
    fs_intersection(s, t).is_disjoint(fs_difference(s, t))
}

/// The union of the finite-set intersection with the left difference is the left finite set.
theorem finite_set_intersection_union_difference_is_self[T](s: FiniteSet[T], t: FiniteSet[T]) {
    fs_union(fs_intersection(s, t), fs_difference(s, t)) = s
}

/// Exact cardinalities for the finite-set intersection and left difference add to the left cardinality.
theorem finite_set_intersection_difference_cardinality_adds[T](s: FiniteSet[T], t: FiniteSet[T],
    n_inter: Nat, n_diff: Nat) {
    fs_intersection(s, t).cardinality_is(n_inter) and fs_difference(s, t).cardinality_is(n_diff)
    implies s.cardinality_is(n_inter + n_diff)
}

/// The finite-set left difference has cardinality equal to the left cardinality minus the intersection cardinality.
theorem finite_set_difference_cardinality_is_sub_intersection[T](s: FiniteSet[T], t: FiniteSet[T],
    n_s: Nat, n_inter: Nat) {
    s.cardinality_is(n_s) and fs_intersection(s, t).cardinality_is(n_inter)
    implies fs_difference(s, t).cardinality_is(n_s - n_inter)
}

/// The finite-set intersection has cardinality equal to the left cardinality minus the left-difference cardinality.
theorem finite_set_intersection_cardinality_is_sub_difference[T](s: FiniteSet[T], t: FiniteSet[T],
    n_s: Nat, n_diff: Nat) {
    s.cardinality_is(n_s) and fs_difference(s, t).cardinality_is(n_diff)
    implies fs_intersection(s, t).cardinality_is(n_s - n_diff)
}

/// Inclusion-exclusion for the union of two finite sets.
theorem finite_set_inclusion_exclusion[T](s: FiniteSet[T], t: FiniteSet[T],
    n_s: Nat, n_t: Nat, n_inter: Nat) {
    s.cardinality_is(n_s) and t.cardinality_is(n_t) and fs_intersection(s, t).cardinality_is(n_inter)
    implies fs_union(s, t).cardinality_is(n_s + n_t - n_inter)
}

/// Inclusion-exclusion for the union of three finite sets.
theorem finite_set_inclusion_exclusion_three[T](
    a: FiniteSet[T], b: FiniteSet[T], c: FiniteSet[T],
    n_a: Nat, n_b: Nat, n_c: Nat,
    n_ab: Nat, n_ac: Nat, n_bc: Nat, n_abc: Nat
) {
    a.cardinality_is(n_a) and b.cardinality_is(n_b) and c.cardinality_is(n_c) and
    fs_intersection(a, b).cardinality_is(n_ab) and
    fs_intersection(a, c).cardinality_is(n_ac) and
    fs_intersection(b, c).cardinality_is(n_bc) and
    fs_intersection(fs_intersection(a, b), c).cardinality_is(n_abc)
    implies fs_union(fs_union(a, b), c).cardinality_is(
        (n_a + n_b - n_ab) + n_c - (n_ac + n_bc - n_abc))
}

/// The sum of singleton cardinalities minus pairwise-intersection cardinalities
/// is a lower bound for the cardinality of a union of three finite sets.
theorem finite_set_bonferroni_lower_three[T](
    a: FiniteSet[T], b: FiniteSet[T], c: FiniteSet[T],
    n_a: Nat, n_b: Nat, n_c: Nat,
    n_ab: Nat, n_ac: Nat, n_bc: Nat, n_union: Nat
) {
    a.cardinality_is(n_a) and b.cardinality_is(n_b) and c.cardinality_is(n_c) and
    fs_intersection(a, b).cardinality_is(n_ab) and
    fs_intersection(a, c).cardinality_is(n_ac) and
    fs_intersection(b, c).cardinality_is(n_bc) and
    fs_union(fs_union(a, b), c).cardinality_is(n_union)
    implies n_a + n_b + n_c - (n_ab + n_ac + n_bc) <= n_union
}

/// Insert a fixed element into a finite subset.
/// This is the map used for the containing half of the subset enumerator.
define insert_into_subset[A](x: A, s: FiniteSet[A]) -> FiniteSet[A] {
    s.insert(x)
}

/// The list of finite subsets of the finite set represented by `items`.
/// Each recursive step keeps each old subset and also inserts the new head.
define list_subsets[A](items: List[A]) -> List[FiniteSet[A]] {
    match items {
        List.nil[A] {
            List.singleton[FiniteSet[A]](FiniteSet.empty[A])
        }
        List.cons[A](head, tail) {
            list_subsets[A](tail) + map[FiniteSet[A], FiniteSet[A]](
                list_subsets[A](tail),
                insert_into_subset[A](head)
            )
        }
    }
}

/// The finite powerset represented by the subset list associated to `items`.
define finite_powerset_from_list[A](items: List[A]) -> FiniteSet[FiniteSet[A]] {
    fs_from_list[FiniteSet[A]](list_subsets[A](items))
}

/// The list-backed finite powerset contains exactly the finite subsets of `FiniteSet.from_list(items)`.
theorem finite_powerset_from_list_exact[A](items: List[A]) {
    forall(s: FiniteSet[A]) {
        finite_powerset_from_list[A](items).contains(s) = s.subset_eq(FiniteSet.from_list[A](items))
    }
}

/// A list-backed finite powerset over a duplicate-free base has cardinality `2 ^ items.length`.
theorem finite_powerset_from_list_cardinality[A](items: List[A]) {
    items.is_unique implies finite_powerset_from_list[A](items).cardinality_is(Nat.2.pow(items.length))
}

/// The sum of `f` over the elements of a finite set.
/// Defined as the sum of `f` mapped over any unique-list representation of the set;
/// well-defined because two unique lists with the same membership give the same mapped sum.
let finite_set_sum[T, A: AddCommMonoid](s: FiniteSet[T], f: T -> A) -> result: A satisfy {
    exists(items: List[T]) {
        fs_from_list(items) = s and items.is_unique and sum[A](map(items, f)) = result
    }
}

/// `finite_set_sum` is computed as the sum of `f` over any unique-list representation.
theorem finite_set_sum_eq_list_sum[T, A: AddCommMonoid](items: List[T], f: T -> A) {
    items.is_unique implies finite_set_sum(fs_from_list(items), f) = sum[A](map(items, f))
}

/// The sum over the empty finite set is zero.
theorem finite_set_sum_empty[T, A: AddCommMonoid](f: T -> A) {
    finite_set_sum(fs_empty[T](Set[T].empty_set), f) = A.0
}

/// Inserting an element not already in the set adds its value to the sum.
theorem finite_set_sum_insert[T, A: AddCommMonoid](s: FiniteSet[T], item: T, f: T -> A) {
    not s.contains(item) implies finite_set_sum(fs_insert(s, item), f) = f(item) + finite_set_sum(s, f)
}

/// Sum is additive in the function argument.
theorem finite_set_sum_add[T, A: AddCommMonoid](s: FiniteSet[T], f: T -> A, g: T -> A) {
    finite_set_sum(s, f) + finite_set_sum(s, g) = finite_set_sum(s, add_fn(f, g))
}

/// The sum over a disjoint union splits as the sum over each part.
theorem finite_set_sum_disjoint_union[T, A: AddCommMonoid](a: FiniteSet[T], b: FiniteSet[T], f: T -> A) {
    a.is_disjoint(b) implies
        finite_set_sum(fs_union(a, b), f) = finite_set_sum(a, f) + finite_set_sum(b, f)
}

/// Scalar multiplication distributes over a finite sum:
/// `c * finite_set_sum(s, f) = finite_set_sum(s, x -> c * f(x))`.
theorem finite_set_sum_scalar_mul[T, S: Semiring](c: S, s: FiniteSet[T], f: T -> S) {
    c * finite_set_sum(s, f) = finite_set_sum(s, mul_fn(c, f))
}
