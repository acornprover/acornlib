from finite_set.base import FiniteSet, finite_set_ext, finite_set_has_unique_list, fs_from_list
from list import List
from finite_set.list_k_subsets import finite_k_subsets_from_list,
    finite_k_subsets_from_list_cardinality_is_length, finite_k_subsets_from_list_exact,
    list_k_subsets
from nat import Nat
from data.basic.set import set_ext

/// The finite family of `k`-element subsets of a finite set.
let finite_k_subsets[T](s: FiniteSet[T], k: Nat) -> result: FiniteSet[FiniteSet[T]] satisfy {
    exists(items: List[T]) {
        fs_from_list(items) = s and items.is_unique and
        result = finite_k_subsets_from_list[T](items, k)
    }
} by {
    finite_set_has_unique_list[T](s)
    let items: List[T] satisfy {
        fs_from_list[T](items) = s and items.is_unique
    }
    let family = finite_k_subsets_from_list[T](items, k)
}

/// The finite k-subset family contains exactly the finite subsets of cardinality `k`.
theorem finite_k_subsets_exact[T](s: FiniteSet[T], k: Nat) {
    forall(t: FiniteSet[T]) {
        finite_k_subsets[T](s, k).contains(t) = (t.subset_eq(s) and t.cardinality_is(k))
    }
} by {
    let items: List[T] satisfy {
        fs_from_list[T](items) = s and items.is_unique and
        finite_k_subsets[T](s, k) = finite_k_subsets_from_list[T](items, k)
    }
    forall(t: FiniteSet[T]) {
        finite_k_subsets_from_list_exact[T](items, k, t)
        t.subset_eq(FiniteSet.from_list[T](items)) = t.subset_eq(s)
        finite_k_subsets[T](s, k).contains(t) = (t.subset_eq(s) and t.cardinality_is(k))
    }
}

/// Any duplicate-free list witness for `s` produces the same finite k-subset family.
theorem finite_k_subsets_from_list_eq_of_unique_list[T](s: FiniteSet[T], k: Nat, items: List[T]) {
    fs_from_list[T](items) = s and items.is_unique implies
        finite_k_subsets[T](s, k) = finite_k_subsets_from_list[T](items, k)
} by {
    if fs_from_list[T](items) = s and items.is_unique {
        forall(t: FiniteSet[T]) {
            finite_k_subsets_exact[T](s, k)
            finite_k_subsets_from_list_exact[T](items, k, t)
            t.subset_eq(FiniteSet.from_list[T](items)) = t.subset_eq(s)
            finite_k_subsets[T](s, k).contains(t) = finite_k_subsets_from_list[T](items, k).contains(t)
            finite_k_subsets[T](s, k).underlying_set.contains(t) =
                finite_k_subsets_from_list[T](items, k).underlying_set.contains(t)
        }
        forall(t: FiniteSet[T]) {
            finite_k_subsets[T](s, k).underlying_set.contains(t) =
                finite_k_subsets_from_list[T](items, k).underlying_set.contains(t)
        }
        set_ext[FiniteSet[T]](
            finite_k_subsets[T](s, k).underlying_set,
            finite_k_subsets_from_list[T](items, k).underlying_set
        )
        finite_set_ext[FiniteSet[T]](finite_k_subsets[T](s, k), finite_k_subsets_from_list[T](items, k))
        finite_k_subsets[T](s, k) = finite_k_subsets_from_list[T](items, k)
    }
}

/// The finite k-subset family is counted by the canonical filtered list for any duplicate-free witness.
theorem finite_k_subsets_cardinality_from_list_length[T](s: FiniteSet[T], k: Nat) {
    forall(items: List[T]) {
        fs_from_list[T](items) = s and items.is_unique implies
        finite_k_subsets[T](s, k).cardinality_is(list_k_subsets[T](items, k).length)
    }
} by {
    forall(items: List[T]) {
        if fs_from_list[T](items) = s and items.is_unique {
            finite_k_subsets_from_list_cardinality_is_length[T](items, k)
            finite_k_subsets_from_list_eq_of_unique_list[T](s, k, items)
            finite_k_subsets[T](s, k).cardinality_is(list_k_subsets[T](items, k).length)
        }
    }
}
