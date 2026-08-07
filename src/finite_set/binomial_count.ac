from finite_set.binomial import finite_k_subsets, finite_k_subsets_cardinality_from_list_length
from finite_set.list_k_subsets import list_k_subsets, list_k_subsets_cons_suc_length
from finite_set.list_subsets import list_subsets, insert_into_subset, list_subsets_exact
from finite_set.base import FiniteSet, fs_from_list, fs_insert, finite_set_ext,
    finite_set_cardinality_has_exact_unique_list,
    finite_set_from_unique_list_cardinality_is_length, finite_set_cardinality_is_well_defined
from data.basic.set import Set, list_set, list_set_contains_eq, set_ext, insert_contains_eq, subset_contains
from list import List, unique_implies_tail_unique, add_length, map, map_filter_length_of_pointwise,
    filter_contained_by_and, filter_false_length
from binary_words import filter_add, unique_cons_not_contains
from combinatorics import Nat, choose_zero, choose_out_of_bounds, pascal_suc_unbounded
numerals Nat

/// A singleton filtered by a predicate true at its member has length one.
lemma singleton_filter_true_length_for_binom[T](item: T, p: T -> Bool) {
    p(item) implies List.singleton[T](item).filter(p).length = Nat.1
} by {
    if p(item) {
        List.cons[T](item, List.nil[T]).filter(p) =
            List.cons[T](item, List.nil[T].filter(p))
        List.nil[T].length = Nat.0
        List.cons[T](item, List.nil[T]).length = Nat.1
        List.singleton[T](item).filter(p).length = Nat.1
    }
}

/// A zero-length list contains no element.
lemma list_length_zero_not_contains_for_binom[T](items: List[T], item: T) {
    items.length = Nat.0 implies not items.contains(item)
} by {
    if items.length = Nat.0 and items.contains(item) {
        match items {
            List.nil[T] {
            }
            List.cons(head, tail) {
                tail.length.suc = Nat.0
                false
            }
        }
    }
}

/// A finite set containing an element cannot have cardinality zero.
lemma finite_set_contains_not_cardinality_zero_for_binom[T](s: FiniteSet[T], item: T) {
    s.contains(item) implies not s.cardinality_is(Nat.0)
} by {
    if s.contains(item) and s.cardinality_is(Nat.0) {
        finite_set_cardinality_has_exact_unique_list[T](s, Nat.0)
        let items: List[T] satisfy {
            fs_from_list[T](items) = s and items.is_unique and items.length = Nat.0
        }
        fs_from_list[T](items).underlying_set = list_set(items)
        list_set(items).contains(item)
        list_set_contains_eq(items, item)
        list_length_zero_not_contains_for_binom[T](items, item)
        false
    }
}

/// Inserting an element makes that element a member of the resulting finite set.
lemma finite_set_insert_contains_for_count[T](s: FiniteSet[T], item: T) {
    s.insert(item).contains(item)
} by {
    fs_insert(s, item).underlying_set = s.underlying_set.insert(item)
    insert_contains_eq(s.underlying_set, item, item)
}

/// A subset of a list-backed set avoids an element not present in the list.
lemma subset_of_from_list_not_contains_for_count[A](head: A, tail: List[A], s: FiniteSet[A]) {
    not tail.contains(head) and s.subset_eq(FiniteSet.from_list[A](tail)) implies not s.contains(head)
} by {
    if not tail.contains(head) and s.subset_eq(FiniteSet.from_list[A](tail)) and s.contains(head) {
        s.underlying_set.contains(head)
        subset_contains(s.underlying_set, FiniteSet.from_list[A](tail).underlying_set, head)
        FiniteSet.from_list[A](tail).underlying_set.contains(head)
        FiniteSet.from_list[A](tail) = fs_from_list[A](tail)
        fs_from_list[A](tail).underlying_set = list_set(tail)
        list_set_contains_eq(tail, head)
        false
    }
}

/// Members of `list_subsets(tail)` avoid a fresh cons head.
lemma list_subsets_tail_members_avoid_head_for_count[A](head: A, tail: List[A], s: FiniteSet[A]) {
    not tail.contains(head) and list_subsets[A](tail).contains(s) implies not s.contains(head)
} by {
    if not tail.contains(head) and list_subsets[A](tail).contains(s) {
        list_subsets_exact[A](tail, s)
        s.subset_eq(FiniteSet.from_list[A](tail))
        subset_of_from_list_not_contains_for_count[A](head, tail, s)
    }
}

/// The filtered zero-subset list is preserved by a fresh cons head.
lemma list_k_subsets_cons_zero_length_for_binom[A](head: A, tail: List[A]) {
    List.cons[A](head, tail).is_unique and list_k_subsets[A](tail, Nat.0).length = Nat.1 implies
        list_k_subsets[A](List.cons[A](head, tail), Nat.0).length = Nat.1
} by {
    if List.cons[A](head, tail).is_unique and list_k_subsets[A](tail, Nat.0).length = Nat.1 {
        unique_cons_not_contains[A](head, tail)
        let left = list_subsets[A](tail)
        let right = map[FiniteSet[A], FiniteSet[A]](left, insert_into_subset[A](head))
        list_k_subsets[A](List.cons[A](head, tail), Nat.0) =
            (left + right).filter(function(s: FiniteSet[A]) { s.cardinality_is(Nat.0) })
        filter_add[FiniteSet[A]](left, right, function(s: FiniteSet[A]) { s.cardinality_is(Nat.0) })
        forall(s: FiniteSet[A]) {
            if left.contains(s) {
                list_subsets_tail_members_avoid_head_for_count[A](head, tail, s)
                finite_set_insert_contains_for_count[A](s, head)
                finite_set_contains_not_cardinality_zero_for_binom[A](s.insert(head), head)
                (function(u: FiniteSet[A]) { u.cardinality_is(Nat.0) })(insert_into_subset[A](head, s)) =
                    (function(u: FiniteSet[A]) { false })(s)
            }
        }
        map_filter_length_of_pointwise[FiniteSet[A], FiniteSet[A]](
            left,
            insert_into_subset[A](head),
            function(u: FiniteSet[A]) { u.cardinality_is(Nat.0) },
            function(u: FiniteSet[A]) { false }
        )
        right.filter(function(s: FiniteSet[A]) { s.cardinality_is(Nat.0) }).length =
            left.filter(function(s: FiniteSet[A]) { false }).length
        filter_false_length[FiniteSet[A]](left)
        add_length[FiniteSet[A]](
            left.filter(function(s: FiniteSet[A]) { s.cardinality_is(Nat.0) }),
            right.filter(function(s: FiniteSet[A]) { s.cardinality_is(Nat.0) })
        )
        list_k_subsets[A](List.cons[A](head, tail), Nat.0).length = Nat.1
    }
}

/// The empty finite set has cardinality zero.
lemma finite_empty_cardinality_is_zero_for_binom[A] {
    FiniteSet.empty[A].cardinality_is(Nat.0)
} by {
    let empty_list = List.nil[A]
    fs_from_list[A](empty_list).underlying_set = list_set(empty_list)
    forall(x: A) {
        not empty_list.contains(x)
        list_set(empty_list).contains(x) = Set.empty_set[A].contains(x)
    }
    set_ext[A](list_set(empty_list), Set.empty_set[A])
    list_set(empty_list) = Set.empty_set[A]
    FiniteSet.empty[A].underlying_set = Set.empty_set[A]
    fs_from_list[A](empty_list).underlying_set = FiniteSet.empty[A].underlying_set
    finite_set_ext[A](fs_from_list[A](empty_list), FiniteSet.empty[A])
    fs_from_list[A](empty_list) = FiniteSet.empty[A]
    empty_list.is_unique
    finite_set_from_unique_list_cardinality_is_length[A](empty_list)
}

/// The empty finite set has no positive cardinality.
lemma finite_empty_not_cardinality_positive_for_binom[A](k: Nat) {
    Nat.0 < k implies not FiniteSet.empty[A].cardinality_is(k)
} by {
    if Nat.0 < k and FiniteSet.empty[A].cardinality_is(k) {
        finite_empty_cardinality_is_zero_for_binom[A]
        finite_set_cardinality_is_well_defined[A](FiniteSet.empty[A], Nat.0, k)
        false
    }
}

/// The filtered list of k-subsets of an empty list has the binomial length.
lemma list_k_subsets_nil_length_binom[A](k: Nat) {
    list_k_subsets[A](List.nil[A], k).length = List.nil[A].length.binom(k)
} by {
    if k = Nat.0 {
        finite_empty_cardinality_is_zero_for_binom[A]
        singleton_filter_true_length_for_binom[FiniteSet[A]](
            FiniteSet.empty[A],
            function(s: FiniteSet[A]) { s.cardinality_is(Nat.0) }
        )
        choose_zero(Nat.0)
        list_k_subsets[A](List.nil[A], Nat.0).length = List.nil[A].length.binom(Nat.0)
    } else {
        Nat.0 < k
        list_k_subsets[A](List.nil[A], k) =
            list_subsets[A](List.nil[A]).filter(function(s: FiniteSet[A]) { s.cardinality_is(k) })
        list_subsets[A](List.nil[A]) = List.singleton[FiniteSet[A]](FiniteSet.empty[A])
        finite_empty_not_cardinality_positive_for_binom[A](k)
        not FiniteSet.empty[A].cardinality_is(k)
        not (function(s: FiniteSet[A]) { s.cardinality_is(k) })(FiniteSet.empty[A])
        let filtered = List.singleton[FiniteSet[A]](FiniteSet.empty[A]).filter(function(s: FiniteSet[A]) { s.cardinality_is(k) })
        if filtered.contains(FiniteSet.empty[A]) {
            filter_contained_by_and[FiniteSet[A]](
                List.singleton[FiniteSet[A]](FiniteSet.empty[A]),
                function(s: FiniteSet[A]) { s.cardinality_is(k) },
                FiniteSet.empty[A]
            )
            false
        }
        if filtered != List.nil[FiniteSet[A]] {
            match filtered {
                List.nil[FiniteSet[A]] {
                }
                List.cons(head, tail) {
                    filtered.contains(head)
                    filter_contained_by_and[FiniteSet[A]](
                        List.singleton[FiniteSet[A]](FiniteSet.empty[A]),
                        function(s: FiniteSet[A]) { s.cardinality_is(k) },
                        head
                    )
                    List.singleton[FiniteSet[A]](FiniteSet.empty[A]).contains(head)
                    head.cardinality_is(k)
                    if head = FiniteSet.empty[A] {
                        false
                    } else {
                        head = FiniteSet.empty[A]
                        false
                    }
                }
            }
        }
        List.singleton[FiniteSet[A]](FiniteSet.empty[A]).filter(function(s: FiniteSet[A]) { s.cardinality_is(k) }).length = Nat.0
        choose_out_of_bounds(Nat.0, k)
        list_k_subsets[A](List.nil[A], k).length = List.nil[A].length.binom(k)
    }
}

/// Predicate used for the all-`k` binomial count induction.
define list_k_subsets_length_binom_pred[A](items: List[A]) -> Bool {
    items.is_unique implies forall(k: Nat) {
        list_k_subsets[A](items, k).length = items.length.binom(k)
    }
}

/// The induction predicate holds for the empty list.
lemma list_k_subsets_length_binom_nil_pred[A] {
    list_k_subsets_length_binom_pred[A](List.nil[A])
} by {
    forall(j: Nat) {
        list_k_subsets_nil_length_binom[A](j)
        list_k_subsets[A](List.nil[A], j).length = List.nil[A].length.binom(j)
    }
}

/// The induction predicate is preserved by adjoining a fresh head.
lemma list_k_subsets_length_binom_cons_pred[A](head: A, tail: List[A]) {
    list_k_subsets_length_binom_pred[A](tail) implies
        list_k_subsets_length_binom_pred[A](List.cons[A](head, tail))
} by {
    if list_k_subsets_length_binom_pred[A](tail) {
        if not list_k_subsets_length_binom_pred[A](List.cons[A](head, tail)) {
            let q: Bool = List.cons[A](head, tail).is_unique implies forall(j: Nat) {
                list_k_subsets[A](List.cons[A](head, tail), j).length =
                    List.cons[A](head, tail).length.binom(j)
            }
            if q {
                false
            }
            if not List.cons[A](head, tail).is_unique {
                false
            }
            unique_implies_tail_unique[A](head, tail)
            tail.is_unique
            list_k_subsets_length_binom_pred[A](tail) = (tail.is_unique implies forall(jj: Nat) {
                list_k_subsets[A](tail, jj).length = tail.length.binom(jj)
            })
            let bad: Nat satisfy {
                not list_k_subsets[A](List.cons[A](head, tail), bad).length =
                    List.cons[A](head, tail).length.binom(bad)
            }
            if bad = Nat.0 {
                list_k_subsets[A](tail, Nat.0).length = tail.length.binom(Nat.0)
                choose_zero(tail.length)
                tail.length.binom(Nat.0) = Nat.1
                list_k_subsets[A](tail, Nat.0).length = Nat.1
                list_k_subsets_cons_zero_length_for_binom[A](head, tail)
                list_k_subsets[A](List.cons[A](head, tail), Nat.0).length = Nat.1
                choose_zero(List.cons[A](head, tail).length)
                List.cons[A](head, tail).length.binom(Nat.0) = Nat.1
                list_k_subsets[A](List.cons[A](head, tail), bad).length =
                    List.cons[A](head, tail).length.binom(bad)
                false
            } else {
                let k0: Nat satisfy { k0.suc = bad }
                list_k_subsets_cons_suc_length[A](head, tail, k0)
                list_k_subsets[A](List.cons[A](head, tail), k0.suc).length =
                    list_k_subsets[A](tail, k0.suc).length + list_k_subsets[A](tail, k0).length
                list_k_subsets[A](tail, k0.suc).length = tail.length.binom(k0.suc)
                list_k_subsets[A](tail, k0).length = tail.length.binom(k0)
                pascal_suc_unbounded(tail.length, k0)
                tail.length.suc.binom(k0.suc) = tail.length.binom(k0) + tail.length.binom(k0.suc)
                tail.length.binom(k0.suc) + tail.length.binom(k0) =
                    tail.length.binom(k0) + tail.length.binom(k0.suc)
                list_k_subsets[A](List.cons[A](head, tail), k0.suc).length =
                    tail.length.suc.binom(k0.suc)
                List.cons[A](head, tail).length = tail.length.suc
                list_k_subsets[A](List.cons[A](head, tail), bad).length =
                    List.cons[A](head, tail).length.binom(bad)
                false
            }
            false
        }
        list_k_subsets_length_binom_pred[A](List.cons[A](head, tail))
    }
}

/// The induction predicate holds for every list.
lemma list_k_subsets_length_binom_pred_all[A] {
    forall(items: List[A]) {
        list_k_subsets_length_binom_pred[A](items)
    }
} by {
    list_k_subsets_length_binom_nil_pred[A]
    forall(head: A, tail: List[A]) {
        if list_k_subsets_length_binom_pred[A](tail) {
            list_k_subsets_length_binom_cons_pred[A](head, tail)
            list_k_subsets_length_binom_pred[A](List.cons[A](head, tail))
        }
    }
    List.induction(function(xs: List[A]) { list_k_subsets_length_binom_pred[A](xs) })
}

/// A duplicate-free list has exactly `items.length choose k` k-element subsets.
theorem list_k_subsets_length_binom[A](items: List[A], k: Nat) {
    items.is_unique implies list_k_subsets[A](items, k).length = items.length.binom(k)
} by {
    list_k_subsets_length_binom_pred_all[A]
    list_k_subsets_length_binom_pred[A](items)
    if items.is_unique {
        list_k_subsets_length_binom_pred[A](items) = (items.is_unique implies forall(j: Nat) {
            list_k_subsets[A](items, j).length = items.length.binom(j)
        })
        list_k_subsets[A](items, k).length = items.length.binom(k)
    }
}

/// A duplicate-free list witness for a finite set gives binomial cardinality
/// for the finite family of k-subsets.
theorem finite_k_subsets_cardinality_is_binom_from_list[T](s: FiniteSet[T], items: List[T], k: Nat) {
    fs_from_list[T](items) = s and items.is_unique implies
        finite_k_subsets[T](s, k).cardinality_is(items.length.binom(k))
} by {
    if fs_from_list[T](items) = s and items.is_unique {
        finite_k_subsets_cardinality_from_list_length[T](s, k)
        finite_k_subsets[T](s, k).cardinality_is(list_k_subsets[T](items, k).length)
        list_k_subsets_length_binom[T](items, k)
        list_k_subsets[T](items, k).length = items.length.binom(k)
        finite_k_subsets[T](s, k).cardinality_is(items.length.binom(k))
    }
}

/// Cardinality-parameter endpoint: a finite set with cardinality `n` has
/// exactly `n choose k` k-element subsets.
theorem finite_k_subsets_cardinality_is_binom[T](s: FiniteSet[T], n: Nat, k: Nat) {
    s.cardinality_is(n) implies finite_k_subsets[T](s, k).cardinality_is(n.binom(k))
} by {
    if s.cardinality_is(n) {
        finite_set_cardinality_has_exact_unique_list[T](s, n)
        let items: List[T] satisfy {
            fs_from_list[T](items) = s and items.is_unique and items.length = n
        }
        finite_k_subsets_cardinality_is_binom_from_list[T](s, items, k)
        finite_k_subsets[T](s, k).cardinality_is(items.length.binom(k))
        items.length.binom(k) = n.binom(k)
        finite_k_subsets[T](s, k).cardinality_is(n.binom(k))
    }
}
