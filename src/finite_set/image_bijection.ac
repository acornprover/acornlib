from finite_set.base import FiniteSet, fs_image, fs_union, finite_set_ext, finite_set_subset_contains,
    finite_set_subset_antisymm, finite_set_intersection_contains_eq,
    finite_set_difference_contains_eq, finite_set_image_monotone, finite_set_image_union,
    finite_set_maps_into_image, finite_set_image_contains_witness,
    finite_set_image_contains_inverse_of_bijection, finite_set_inverse_contains_imp_image_contains,
    finite_set_image_cardinality_is_of_injective, finite_set_cardinality_is_well_defined
from data.basic.functions import Inhabited, inverse_fn, is_injective_fn, is_bijection_fn,
    bijection_fn_is_injective, bijection_fn_is_surjective, injective_fn_eq,
    inverse_fn_apply_of_surjective, inverse_fn_apply_image_of_bijection,
    inverse_fn_is_bijection_of_bijection
from nat import Nat
from data.basic.set import set_ext, subset_contains_eq, cardinality_always_exists

numerals Nat

/// True if two maps agree on every member of a finite set.
define finite_set_maps_agree_on[T, U](s: FiniteSet[T], f: T -> U, g: T -> U) -> Bool {
    forall(x: T) {
        s.contains(x) implies f(x) = g(x)
    }
}

/// Source-restricted equality of maps gives equality of finite-set images.
theorem finite_set_image_eq_of_maps_agree_on[T, U](s: FiniteSet[T], f: T -> U, g: T -> U) {
    finite_set_maps_agree_on(s, f, g) implies fs_image(s, f) = fs_image(s, g)
} by {
    let left = fs_image(s, f)
    let right = fs_image(s, g)
    if finite_set_maps_agree_on(s, f, g) {
        finite_set_maps_agree_on(s, f, g) = forall(x: T) {
            s.contains(x) implies f(x) = g(x)
        }
        forall(y: U) {
            if left.underlying_set.contains(y) {
                finite_set_image_contains_witness(s, f, y)
                let x: T satisfy {
                    s.contains(x) and y = f(x)
                }
                y = g(x)
                finite_set_maps_into_image(s, g, x)
                right.underlying_set.contains(y)
            }
            if right.underlying_set.contains(y) {
                finite_set_image_contains_witness(s, g, y)
                let x: T satisfy {
                    s.contains(x) and y = g(x)
                }
                y = f(x)
                finite_set_maps_into_image(s, f, x)
                left.underlying_set.contains(y)
            }
            left.underlying_set.contains(y) = right.underlying_set.contains(y)
        }
        set_ext(left.underlying_set, right.underlying_set)
        finite_set_ext(left, right)
        left = right
    }
}

/// Source-restricted equality on a finite superset gives equal images of each finite subset.
theorem finite_set_image_eq_of_maps_agree_on_superset[T, U](a: FiniteSet[T], s: FiniteSet[T],
    f: T -> U, g: T -> U) {
    a.subset_eq(s) and finite_set_maps_agree_on(s, f, g) implies fs_image(a, f) = fs_image(a, g)
} by {
    if a.subset_eq(s) and finite_set_maps_agree_on(s, f, g) {
        finite_set_maps_agree_on(s, f, g) = forall(x: T) {
            s.contains(x) implies f(x) = g(x)
        }
        forall(x: T) {
            if a.contains(x) {
                finite_set_subset_contains(a, s, x)
                f(x) = g(x)
            }
        }
        finite_set_maps_agree_on(a, f, g)
        finite_set_image_eq_of_maps_agree_on(a, f, g)
        fs_image(a, f) = fs_image(a, g)
    }
}

/// Exact cardinality of a finite-set image depends only on the map values on the source.
theorem finite_set_image_cardinality_is_iff_of_maps_agree_on[T, U](s: FiniteSet[T],
    f: T -> U, g: T -> U, n: Nat) {
    finite_set_maps_agree_on(s, f, g) implies
        fs_image(s, f).cardinality_is(n) = fs_image(s, g).cardinality_is(n)
} by {
    if finite_set_maps_agree_on(s, f, g) {
        finite_set_image_eq_of_maps_agree_on(s, f, g)
        if fs_image(s, f).cardinality_is(n) {
            fs_image(s, g).cardinality_is(n)
        }
        if fs_image(s, g).cardinality_is(n) {
            fs_image(s, f).cardinality_is(n)
        }
        fs_image(s, f).cardinality_is(n) = fs_image(s, g).cardinality_is(n)
    }
}

/// If a map agrees on a finite source with an injective map, its image has the source cardinality.
theorem finite_set_image_cardinality_is_of_agreeing_injective[T, U](s: FiniteSet[T],
    f: T -> U, g: T -> U, n: Nat) {
    is_injective_fn(f) and finite_set_maps_agree_on(s, f, g) and s.cardinality_is(n)
    implies fs_image(s, g).cardinality_is(n)
} by {
    if is_injective_fn(f) and finite_set_maps_agree_on(s, f, g) and s.cardinality_is(n) {
        finite_set_image_cardinality_is_of_injective(s, f, n)
        finite_set_image_cardinality_is_iff_of_maps_agree_on(s, f, g, n)
        fs_image(s, g).cardinality_is(n)
    }
}

/// The finite preimage of a finite target along a bijection, represented as the image
/// under the chosen inverse.
define finite_set_preimage_of_bijection[T: Inhabited, U](t: FiniteSet[U], f: T -> U) ->
    FiniteSet[T] {
    fs_image(t, inverse_fn(f))
}

/// A finite-set image is contained in a finite target exactly when every source member maps into it.
theorem finite_set_image_subset_iff_maps_into[T, U](s: FiniteSet[T], t: FiniteSet[U],
    f: T -> U) {
    fs_image(s, f).subset_eq(t) = forall(x: T) {
        s.contains(x) implies t.contains(f(x))
    }
} by {
    if fs_image(s, f).subset_eq(t) {
        forall(x: T) {
            if s.contains(x) {
                finite_set_maps_into_image(s, f, x)
                finite_set_subset_contains(fs_image(s, f), t, f(x))
                t.contains(f(x))
            }
        }
    }
    if forall(x: T) { s.contains(x) implies t.contains(f(x)) } {
        forall(y: U) {
            if fs_image(s, f).underlying_set.contains(y) {
                finite_set_image_contains_witness(s, f, y)
                let x: T satisfy {
                    s.contains(x) and y = f(x)
                }
                t.contains(y)
                t.underlying_set.contains(y)
            }
        }
        subset_contains_eq(fs_image(s, f).underlying_set, t.underlying_set)
        forall(y: U) {
            fs_image(s, f).underlying_set.contains(y) implies t.underlying_set.contains(y)
        }
        fs_image(s, f).underlying_set.subset(t.underlying_set)
        fs_image(s, f).subset_eq(t)
    }
}

/// If two maps agree on a finite source, image containment may be checked using either map.
theorem finite_set_image_subset_iff_maps_into_agree_on[T, U](s: FiniteSet[T], t: FiniteSet[U],
    f: T -> U, g: T -> U) {
    finite_set_maps_agree_on(s, f, g) implies fs_image(s, f).subset_eq(t) = forall(x: T) {
        s.contains(x) implies t.contains(g(x))
    }
} by {
    if finite_set_maps_agree_on(s, f, g) {
        finite_set_image_eq_of_maps_agree_on(s, f, g)
        fs_image(s, f) = fs_image(s, g)
        finite_set_image_subset_iff_maps_into(s, t, g)
        fs_image(s, f).subset_eq(t) = forall(x: T) {
            s.contains(x) implies t.contains(g(x))
        }
    }
}

/// A finite target is contained in a bijective finite-set image exactly when every target
/// member has its chosen inverse in the source finite set.
theorem finite_set_subset_image_iff_inverse_contains_of_bijection[T: Inhabited, U](
    s: FiniteSet[T], t: FiniteSet[U], f: T -> U) {
    is_bijection_fn(f) implies t.subset_eq(fs_image(s, f)) = forall(y: U) {
        t.contains(y) implies s.contains(inverse_fn(f, y))
    }
} by {
    if is_bijection_fn(f) {
        if t.subset_eq(fs_image(s, f)) {
            forall(y: U) {
                if t.contains(y) {
                    finite_set_subset_contains(t, fs_image(s, f), y)
                    finite_set_image_contains_inverse_of_bijection(s, f, y)
                    s.contains(inverse_fn(f, y))
                }
            }
        }
        if forall(y: U) { t.contains(y) implies s.contains(inverse_fn(f, y)) } {
            forall(y: U) {
                if t.underlying_set.contains(y) {
                    finite_set_inverse_contains_imp_image_contains(s, f, y)
                    fs_image(s, f).contains(y)
                    fs_image(s, f).underlying_set.contains(y)
                }
            }
            subset_contains_eq(t.underlying_set, fs_image(s, f).underlying_set)
            forall(y: U) {
                t.underlying_set.contains(y) implies fs_image(s, f).underlying_set.contains(y)
            }
            t.underlying_set.subset(fs_image(s, f).underlying_set)
            t.subset_eq(fs_image(s, f))
        }
        t.subset_eq(fs_image(s, f)) = forall(y: U) {
            t.contains(y) implies s.contains(inverse_fn(f, y))
        }
    }
}

/// Under injectivity, inclusion of finite-set images reflects inclusion of source finite sets.
theorem finite_set_image_subset_imp_subset_of_injective[T, U](a: FiniteSet[T], b: FiniteSet[T],
    f: T -> U) {
    is_injective_fn(f) and fs_image(a, f).subset_eq(fs_image(b, f)) implies a.subset_eq(b)
} by {
    if is_injective_fn(f) and fs_image(a, f).subset_eq(fs_image(b, f)) {
        forall(x: T) {
            if a.underlying_set.contains(x) {
                finite_set_maps_into_image(a, f, x)
                finite_set_subset_contains(fs_image(a, f), fs_image(b, f), f(x))
                finite_set_image_contains_witness(b, f, f(x))
                let x2: T satisfy {
                    b.contains(x2) and f(x) = f(x2)
                }
                injective_fn_eq(f, x, x2)
                x = x2
                b.underlying_set.contains(x)
            }
        }
        subset_contains_eq(a.underlying_set, b.underlying_set)
        forall(x: T) {
            a.underlying_set.contains(x) implies b.underlying_set.contains(x)
        }
        a.subset_eq(b)
    }
}

/// Under injectivity, inclusion of finite-set images is equivalent to inclusion of sources.
theorem finite_set_image_subset_iff_of_injective[T, U](a: FiniteSet[T], b: FiniteSet[T],
    f: T -> U) {
    is_injective_fn(f) implies fs_image(a, f).subset_eq(fs_image(b, f)) = a.subset_eq(b)
} by {
    if is_injective_fn(f) {
        if fs_image(a, f).subset_eq(fs_image(b, f)) {
            finite_set_image_subset_imp_subset_of_injective(a, b, f)
            a.subset_eq(b)
        }
        if a.subset_eq(b) {
            finite_set_image_monotone(a, b, f)
            fs_image(a, f).subset_eq(fs_image(b, f))
        }
        fs_image(a, f).subset_eq(fs_image(b, f)) = a.subset_eq(b)
    }
}

/// Under injectivity, equal finite-set images come from equal finite source sets.
theorem finite_set_image_eq_imp_eq_of_injective[T, U](a: FiniteSet[T], b: FiniteSet[T],
    f: T -> U) {
    is_injective_fn(f) and fs_image(a, f) = fs_image(b, f) implies a = b
} by {
    if is_injective_fn(f) and fs_image(a, f) = fs_image(b, f) {
        finite_set_image_subset_imp_subset_of_injective(a, b, f)
        fs_image(b, f).subset_eq(fs_image(a, f))
        finite_set_image_subset_imp_subset_of_injective(b, a, f)
        b.subset_eq(a)
        finite_set_subset_antisymm(a, b)
        a = b
    }
}

/// Under injectivity, equality of finite-set images is equivalent to equality of sources.
theorem finite_set_image_eq_iff_of_injective[T, U](a: FiniteSet[T], b: FiniteSet[T],
    f: T -> U) {
    is_injective_fn(f) implies (fs_image(a, f) = fs_image(b, f)) = (a = b)
} by {
    if is_injective_fn(f) {
        if fs_image(a, f) = fs_image(b, f) {
            finite_set_image_eq_imp_eq_of_injective(a, b, f)
            a = b
        }
        if a = b {
            fs_image(a, f) = fs_image(b, f)
        }
        (fs_image(a, f) = fs_image(b, f)) = (a = b)
    }
}

/// Under injectivity, finite-set image preserves intersections.
theorem finite_set_image_intersection_of_injective[T, U](a: FiniteSet[T], b: FiniteSet[T],
    f: T -> U) {
    is_injective_fn(f) implies fs_image(a.intersection(b), f) =
        fs_image(a, f).intersection(fs_image(b, f))
} by {
    let left = fs_image(a.intersection(b), f)
    let right = fs_image(a, f).intersection(fs_image(b, f))
    if is_injective_fn(f) {
        finite_set_image_subset_iff_maps_into(a.intersection(b), right, f)
        forall(x: T) {
            if a.intersection(b).contains(x) {
                finite_set_intersection_contains_eq(a, b, x)
                a.contains(x)
                b.contains(x)
                finite_set_maps_into_image(a, f, x)
                finite_set_maps_into_image(b, f, x)
                fs_image(a, f).contains(f(x))
                fs_image(b, f).contains(f(x))
                finite_set_intersection_contains_eq(fs_image(a, f), fs_image(b, f), f(x))
                right.contains(f(x))
            }
        }
        left.subset_eq(right)

        forall(y: U) {
            if right.underlying_set.contains(y) {
                right.contains(y)
                finite_set_intersection_contains_eq(fs_image(a, f), fs_image(b, f), y)
                fs_image(a, f).contains(y)
                finite_set_image_contains_witness(a, f, y)
                let xa: T satisfy {
                    a.contains(xa) and y = f(xa)
                }
                finite_set_image_contains_witness(b, f, y)
                let xb: T satisfy {
                    b.contains(xb) and y = f(xb)
                }
                injective_fn_eq(f, xa, xb)
                xa = xb
                finite_set_intersection_contains_eq(a, b, xa)
                a.intersection(b).contains(xa)
                finite_set_maps_into_image(a.intersection(b), f, xa)
                left.underlying_set.contains(y)
            }
        }
        subset_contains_eq(right.underlying_set, left.underlying_set)
        forall(y: U) {
            right.underlying_set.contains(y) implies left.underlying_set.contains(y)
        }
        right.subset_eq(left)

        finite_set_subset_antisymm(left, right)
        left = right
    }
}

/// Under injectivity, finite-set image preserves differences.
theorem finite_set_image_difference_of_injective[T, U](a: FiniteSet[T], b: FiniteSet[T],
    f: T -> U) {
    is_injective_fn(f) implies fs_image(a.difference(b), f) =
        fs_image(a, f).difference(fs_image(b, f))
} by {
    let left = fs_image(a.difference(b), f)
    let right = fs_image(a, f).difference(fs_image(b, f))
    if is_injective_fn(f) {
        finite_set_image_subset_iff_maps_into(a.difference(b), right, f)
        forall(x: T) {
            if a.difference(b).contains(x) {
                finite_set_difference_contains_eq(a, b, x)
                a.contains(x)
                not b.contains(x)
                finite_set_maps_into_image(a, f, x)
                fs_image(a, f).contains(f(x))
                if fs_image(b, f).contains(f(x)) {
                    finite_set_image_contains_witness(b, f, f(x))
                    let xb: T satisfy {
                        b.contains(xb) and f(x) = f(xb)
                    }
                    injective_fn_eq(f, x, xb)
                    x = xb
                    false
                }
                not fs_image(b, f).contains(f(x))
                finite_set_difference_contains_eq(fs_image(a, f), fs_image(b, f), f(x))
                right.contains(f(x))
            }
        }
        left.subset_eq(right)

        forall(y: U) {
            if right.underlying_set.contains(y) {
                right.contains(y)
                finite_set_difference_contains_eq(fs_image(a, f), fs_image(b, f), y)
                fs_image(a, f).contains(y)
                not fs_image(b, f).contains(y)
                finite_set_image_contains_witness(a, f, y)
                let x: T satisfy {
                    a.contains(x) and y = f(x)
                }
                if b.contains(x) {
                    finite_set_maps_into_image(b, f, x)
                    false
                }
                finite_set_difference_contains_eq(a, b, x)
                a.difference(b).contains(x)
                finite_set_maps_into_image(a.difference(b), f, x)
                left.underlying_set.contains(y)
            }
        }
        subset_contains_eq(right.underlying_set, left.underlying_set)
        forall(y: U) {
            right.underlying_set.contains(y) implies left.underlying_set.contains(y)
        }
        right.subset_eq(left)

        finite_set_subset_antisymm(left, right)
        left = right
    }
}

/// Under a bijection, inclusion of finite-set images is equivalent to inclusion of sources.
theorem finite_set_image_subset_iff_of_bijection[T, U](a: FiniteSet[T], b: FiniteSet[T],
    f: T -> U) {
    is_bijection_fn(f) implies fs_image(a, f).subset_eq(fs_image(b, f)) = a.subset_eq(b)
} by {
    if is_bijection_fn(f) {
        bijection_fn_is_injective(f)
        finite_set_image_subset_iff_of_injective(a, b, f)
        fs_image(a, f).subset_eq(fs_image(b, f)) = a.subset_eq(b)
    }
}

/// Under a bijection, equality of finite-set images is equivalent to equality of sources.
theorem finite_set_image_eq_iff_of_bijection[T, U](a: FiniteSet[T], b: FiniteSet[T],
    f: T -> U) {
    is_bijection_fn(f) implies (fs_image(a, f) = fs_image(b, f)) = (a = b)
} by {
    if is_bijection_fn(f) {
        bijection_fn_is_injective(f)
        finite_set_image_eq_iff_of_injective(a, b, f)
        (fs_image(a, f) = fs_image(b, f)) = (a = b)
    }
}

/// If an injective image of a finite set has exact cardinality `n`, then the source has
/// exact cardinality `n`.
theorem finite_set_cardinality_is_of_injective_image[T, U](s: FiniteSet[T], f: T -> U, n: Nat) {
    is_injective_fn(f) and fs_image(s, f).cardinality_is(n) implies s.cardinality_is(n)
} by {
    if is_injective_fn(f) and fs_image(s, f).cardinality_is(n) {
        s.underlying_set.is_finite
        cardinality_always_exists(s.underlying_set)
        let m: Nat satisfy {
            s.underlying_set.cardinality_is(m)
        }
        s.cardinality_is(m)
        finite_set_image_cardinality_is_of_injective(s, f, m)
        fs_image(s, f).cardinality_is(m)
        finite_set_cardinality_is_well_defined(fs_image(s, f), n, m)
        n = m
        s.cardinality_is(n)
    }
}

/// For an injective map, exact cardinality is transported both ways between a finite set
/// and its image.
theorem finite_set_image_cardinality_is_iff_of_injective[T, U](s: FiniteSet[T], f: T -> U,
    n: Nat) {
    is_injective_fn(f) implies fs_image(s, f).cardinality_is(n) = s.cardinality_is(n)
} by {
    if is_injective_fn(f) {
        if fs_image(s, f).cardinality_is(n) {
            finite_set_cardinality_is_of_injective_image(s, f, n)
            s.cardinality_is(n)
        }
        if s.cardinality_is(n) {
            finite_set_image_cardinality_is_of_injective(s, f, n)
            fs_image(s, f).cardinality_is(n)
        }
        fs_image(s, f).cardinality_is(n) = s.cardinality_is(n)
    }
}

/// For a bijection, exact cardinality is transported both ways between a finite set and
/// its image.
theorem finite_set_image_cardinality_is_iff_of_bijection[T, U](s: FiniteSet[T], f: T -> U,
    n: Nat) {
    is_bijection_fn(f) implies fs_image(s, f).cardinality_is(n) = s.cardinality_is(n)
} by {
    if is_bijection_fn(f) {
        bijection_fn_is_injective(f)
        is_injective_fn(f)
        finite_set_image_cardinality_is_iff_of_injective(s, f, n)
        fs_image(s, f).cardinality_is(n) = s.cardinality_is(n)
    }
}

/// Membership in the finite preimage along a bijection is membership of the forward
/// image in the target finite set.
theorem finite_set_preimage_of_bijection_contains_eq[T: Inhabited, U](t: FiniteSet[U],
    f: T -> U, x: T) {
    is_bijection_fn(f) implies finite_set_preimage_of_bijection(t, f).contains(x) =
        t.contains(f(x))
} by {
    if is_bijection_fn(f) {
        if finite_set_preimage_of_bijection(t, f).contains(x) {
            finite_set_preimage_of_bijection(t, f) = fs_image(t, inverse_fn(f))
            fs_image(t, inverse_fn(f)).contains(x)
            finite_set_image_contains_witness(t, inverse_fn(f), x)
            let y: U satisfy {
                t.contains(y) and x = inverse_fn(f, y)
            }
            bijection_fn_is_surjective(f)
            inverse_fn_apply_of_surjective(f, y)
            f(inverse_fn(f, y)) = y
            f(x) = y
            t.contains(f(x))
        }
        if t.contains(f(x)) {
            finite_set_maps_into_image(t, inverse_fn(f), f(x))
            fs_image(t, inverse_fn(f)).contains(inverse_fn(f, f(x)))
            inverse_fn_apply_image_of_bijection(f, x)
            inverse_fn(f, f(x)) = x
            finite_set_preimage_of_bijection(t, f) = fs_image(t, inverse_fn(f))
            finite_set_preimage_of_bijection(t, f).contains(x)
        }
        finite_set_preimage_of_bijection(t, f).contains(x) = t.contains(f(x))
    }
}

/// A finite source is contained in a bijective finite preimage exactly when every source
/// member maps into the target finite set.
theorem finite_set_subset_preimage_iff_maps_into_of_bijection[T: Inhabited, U](
    s: FiniteSet[T], t: FiniteSet[U], f: T -> U) {
    is_bijection_fn(f) implies s.subset_eq(finite_set_preimage_of_bijection(t, f)) =
        forall(x: T) {
            s.contains(x) implies t.contains(f(x))
        }
} by {
    let p = finite_set_preimage_of_bijection(t, f)
    if is_bijection_fn(f) {
        if s.subset_eq(p) {
            forall(x: T) {
                if s.contains(x) {
                    finite_set_subset_contains(s, p, x)
                    p.contains(x)
                    finite_set_preimage_of_bijection_contains_eq(t, f, x)
                    t.contains(f(x))
                }
            }
        }
        if forall(x: T) { s.contains(x) implies t.contains(f(x)) } {
            forall(x: T) {
                if s.underlying_set.contains(x) {
                    s.contains(x)
                    t.contains(f(x))
                    finite_set_preimage_of_bijection_contains_eq(t, f, x)
                    p.contains(x)
                    p.underlying_set.contains(x)
                }
            }
            subset_contains_eq(s.underlying_set, p.underlying_set)
            forall(x: T) {
                s.underlying_set.contains(x) implies p.underlying_set.contains(x)
            }
            s.underlying_set.subset(p.underlying_set)
            s.subset_eq(p)
        }
        s.subset_eq(p) = forall(x: T) {
            s.contains(x) implies t.contains(f(x))
        }
    }
}

/// For a bijection, finite-set image inclusion is equivalent to inclusion in the
/// corresponding finite preimage.
theorem finite_set_image_subset_iff_subset_preimage_of_bijection[T: Inhabited, U](
    s: FiniteSet[T], t: FiniteSet[U], f: T -> U) {
    is_bijection_fn(f) implies fs_image(s, f).subset_eq(t) =
        s.subset_eq(finite_set_preimage_of_bijection(t, f))
} by {
    if is_bijection_fn(f) {
        finite_set_image_subset_iff_maps_into(s, t, f)
        finite_set_subset_preimage_iff_maps_into_of_bijection(s, t, f)
        fs_image(s, f).subset_eq(t) = forall(x: T) {
            s.contains(x) implies t.contains(f(x))
        }
        s.subset_eq(finite_set_preimage_of_bijection(t, f)) = forall(x: T) {
            s.contains(x) implies t.contains(f(x))
        }
        fs_image(s, f).subset_eq(t) = s.subset_eq(finite_set_preimage_of_bijection(t, f))
    }
}

/// The image of the finite preimage along a bijection is the original finite target.
theorem finite_set_image_preimage_of_bijection[T: Inhabited, U](t: FiniteSet[U], f: T -> U) {
    is_bijection_fn(f) implies fs_image(finite_set_preimage_of_bijection(t, f), f) = t
} by {
    let p = finite_set_preimage_of_bijection(t, f)
    if is_bijection_fn(f) {
        finite_set_image_subset_iff_maps_into(p, t, f)
        forall(x: T) {
            if p.contains(x) {
                finite_set_preimage_of_bijection_contains_eq(t, f, x)
                t.contains(f(x))
            }
        }
        fs_image(p, f).subset_eq(t)

        finite_set_subset_image_iff_inverse_contains_of_bijection(p, t, f)
        forall(y: U) {
            if t.contains(y) {
                p = fs_image(t, inverse_fn(f))
                finite_set_maps_into_image(t, inverse_fn(f), y)
                fs_image(t, inverse_fn(f)).contains(inverse_fn(f, y))
                p.contains(inverse_fn(f, y))
            }
        }
        t.subset_eq(fs_image(p, f))

        finite_set_subset_antisymm(fs_image(p, f), t)
        fs_image(p, f) = t
    }
}

/// The finite preimage of a finite-set image along a bijection is the original finite source.
theorem finite_set_preimage_image_of_bijection[T: Inhabited, U](s: FiniteSet[T], f: T -> U) {
    is_bijection_fn(f) implies finite_set_preimage_of_bijection(fs_image(s, f), f) = s
} by {
    let p = finite_set_preimage_of_bijection(fs_image(s, f), f)
    if is_bijection_fn(f) {
        forall(x: T) {
            if p.underlying_set.contains(x) {
                p.contains(x)
                finite_set_preimage_of_bijection_contains_eq(fs_image(s, f), f, x)
                fs_image(s, f).contains(f(x))
                finite_set_image_contains_witness(s, f, f(x))
                let x2: T satisfy {
                    s.contains(x2) and f(x) = f(x2)
                }
                bijection_fn_is_injective(f)
                injective_fn_eq(f, x, x2)
                x = x2
                s.contains(x)
                s.underlying_set.contains(x)
            }
        }
        subset_contains_eq(p.underlying_set, s.underlying_set)
        forall(x: T) {
            p.underlying_set.contains(x) implies s.underlying_set.contains(x)
        }
        p.underlying_set.subset(s.underlying_set)
        p.subset_eq(s)

        forall(x: T) {
            if s.underlying_set.contains(x) {
                s.contains(x)
                finite_set_maps_into_image(s, f, x)
                fs_image(s, f).contains(f(x))
                finite_set_preimage_of_bijection_contains_eq(fs_image(s, f), f, x)
                p.contains(x)
                p.underlying_set.contains(x)
            }
        }
        subset_contains_eq(s.underlying_set, p.underlying_set)
        forall(x: T) {
            s.underlying_set.contains(x) implies p.underlying_set.contains(x)
        }
        s.underlying_set.subset(p.underlying_set)
        s.subset_eq(p)

        finite_set_subset_antisymm(p, s)
        p = s
    }
}

/// For a bijection, inclusion of finite preimages is equivalent to inclusion of finite targets.
theorem finite_set_preimage_subset_iff_of_bijection[T: Inhabited, U](a: FiniteSet[U],
    b: FiniteSet[U], f: T -> U) {
    is_bijection_fn(f) implies finite_set_preimage_of_bijection(a, f).subset_eq(
        finite_set_preimage_of_bijection(b, f)) = a.subset_eq(b)
} by {
    if is_bijection_fn(f) {
        inverse_fn_is_bijection_of_bijection(f)
        is_bijection_fn(inverse_fn(f))
        finite_set_image_subset_iff_of_bijection(a, b, inverse_fn(f))
        fs_image(a, inverse_fn(f)).subset_eq(fs_image(b, inverse_fn(f))) = a.subset_eq(b)
        finite_set_preimage_of_bijection(a, f) = fs_image(a, inverse_fn(f))
        finite_set_preimage_of_bijection(b, f) = fs_image(b, inverse_fn(f))
        finite_set_preimage_of_bijection(a, f).subset_eq(
            finite_set_preimage_of_bijection(b, f)) = a.subset_eq(b)
    }
}

/// For a bijection, equality of finite preimages is equivalent to equality of finite targets.
theorem finite_set_preimage_eq_iff_of_bijection[T: Inhabited, U](a: FiniteSet[U],
    b: FiniteSet[U], f: T -> U) {
    is_bijection_fn(f) implies (finite_set_preimage_of_bijection(a, f) =
        finite_set_preimage_of_bijection(b, f)) = (a = b)
} by {
    if is_bijection_fn(f) {
        inverse_fn_is_bijection_of_bijection(f)
        is_bijection_fn(inverse_fn(f))
        finite_set_image_eq_iff_of_bijection(a, b, inverse_fn(f))
        (fs_image(a, inverse_fn(f)) = fs_image(b, inverse_fn(f))) = (a = b)
        finite_set_preimage_of_bijection(a, f) = fs_image(a, inverse_fn(f))
        finite_set_preimage_of_bijection(b, f) = fs_image(b, inverse_fn(f))
        (finite_set_preimage_of_bijection(a, f) = finite_set_preimage_of_bijection(b, f)) =
            (a = b)
    }
}

/// For a bijection, exact cardinality is transported both ways between a finite target and
/// its finite preimage.
theorem finite_set_preimage_cardinality_is_iff_of_bijection[T: Inhabited, U](t: FiniteSet[U],
    f: T -> U, n: Nat) {
    is_bijection_fn(f) implies finite_set_preimage_of_bijection(t, f).cardinality_is(n) =
        t.cardinality_is(n)
} by {
    if is_bijection_fn(f) {
        inverse_fn_is_bijection_of_bijection(f)
        is_bijection_fn(inverse_fn(f))
        finite_set_image_cardinality_is_iff_of_bijection(t, inverse_fn(f), n)
        fs_image(t, inverse_fn(f)).cardinality_is(n) = t.cardinality_is(n)
        finite_set_preimage_of_bijection(t, f) = fs_image(t, inverse_fn(f))
        finite_set_preimage_of_bijection(t, f).cardinality_is(n) = t.cardinality_is(n)
    }
}

/// Finite preimage along a bijection preserves finite unions.
theorem finite_set_preimage_union_of_bijection[T: Inhabited, U](a: FiniteSet[U],
    b: FiniteSet[U], f: T -> U) {
    is_bijection_fn(f) implies finite_set_preimage_of_bijection(a.union(b), f) =
        finite_set_preimage_of_bijection(a, f).union(finite_set_preimage_of_bijection(b, f))
} by {
    if is_bijection_fn(f) {
        a.union(b) = fs_union(a, b)
        finite_set_image_union(a, b, inverse_fn(f))
        fs_image(a.union(b), inverse_fn(f)) =
            fs_image(a, inverse_fn(f)).union(fs_image(b, inverse_fn(f)))
        finite_set_preimage_of_bijection(a.union(b), f) = fs_image(a.union(b), inverse_fn(f))
        finite_set_preimage_of_bijection(a, f) = fs_image(a, inverse_fn(f))
        finite_set_preimage_of_bijection(b, f) = fs_image(b, inverse_fn(f))
        finite_set_preimage_of_bijection(a, f).union(finite_set_preimage_of_bijection(b, f)) =
            fs_image(a, inverse_fn(f)).union(fs_image(b, inverse_fn(f)))
        finite_set_preimage_of_bijection(a.union(b), f) =
            finite_set_preimage_of_bijection(a, f).union(finite_set_preimage_of_bijection(b, f))
    }
}

/// Finite preimage along a bijection preserves finite intersections.
theorem finite_set_preimage_intersection_of_bijection[T: Inhabited, U](a: FiniteSet[U],
    b: FiniteSet[U], f: T -> U) {
    is_bijection_fn(f) implies finite_set_preimage_of_bijection(a.intersection(b), f) =
        finite_set_preimage_of_bijection(a, f).intersection(finite_set_preimage_of_bijection(b, f))
} by {
    if is_bijection_fn(f) {
        inverse_fn_is_bijection_of_bijection(f)
        is_bijection_fn(inverse_fn(f))
        bijection_fn_is_injective(inverse_fn(f))
        is_injective_fn(inverse_fn(f))
        finite_set_image_intersection_of_injective(a, b, inverse_fn(f))
        fs_image(a.intersection(b), inverse_fn(f)) =
            fs_image(a, inverse_fn(f)).intersection(fs_image(b, inverse_fn(f)))
        finite_set_preimage_of_bijection(a.intersection(b), f) =
            fs_image(a.intersection(b), inverse_fn(f))
        finite_set_preimage_of_bijection(a, f) = fs_image(a, inverse_fn(f))
        finite_set_preimage_of_bijection(b, f) = fs_image(b, inverse_fn(f))
        finite_set_preimage_of_bijection(a, f).intersection(finite_set_preimage_of_bijection(b, f)) =
            fs_image(a, inverse_fn(f)).intersection(fs_image(b, inverse_fn(f)))
        finite_set_preimage_of_bijection(a.intersection(b), f) =
            finite_set_preimage_of_bijection(a, f).intersection(finite_set_preimage_of_bijection(b, f))
    }
}

/// Finite preimage along a bijection preserves finite differences.
theorem finite_set_preimage_difference_of_bijection[T: Inhabited, U](a: FiniteSet[U],
    b: FiniteSet[U], f: T -> U) {
    is_bijection_fn(f) implies finite_set_preimage_of_bijection(a.difference(b), f) =
        finite_set_preimage_of_bijection(a, f).difference(finite_set_preimage_of_bijection(b, f))
} by {
    if is_bijection_fn(f) {
        inverse_fn_is_bijection_of_bijection(f)
        is_bijection_fn(inverse_fn(f))
        bijection_fn_is_injective(inverse_fn(f))
        is_injective_fn(inverse_fn(f))
        finite_set_image_difference_of_injective(a, b, inverse_fn(f))
        fs_image(a.difference(b), inverse_fn(f)) =
            fs_image(a, inverse_fn(f)).difference(fs_image(b, inverse_fn(f)))
        finite_set_preimage_of_bijection(a.difference(b), f) =
            fs_image(a.difference(b), inverse_fn(f))
        finite_set_preimage_of_bijection(a, f) = fs_image(a, inverse_fn(f))
        finite_set_preimage_of_bijection(b, f) = fs_image(b, inverse_fn(f))
        finite_set_preimage_of_bijection(a, f).difference(finite_set_preimage_of_bijection(b, f)) =
            fs_image(a, inverse_fn(f)).difference(fs_image(b, inverse_fn(f)))
        finite_set_preimage_of_bijection(a.difference(b), f) =
            finite_set_preimage_of_bijection(a, f).difference(finite_set_preimage_of_bijection(b, f))
    }
}

/// A finite subset of a bijective image pulls back to a finite subset of the source.
theorem finite_set_preimage_subset_source_of_subset_image_bijection[T: Inhabited, U](
    s: FiniteSet[T], t: FiniteSet[U], f: T -> U) {
    is_bijection_fn(f) and t.subset_eq(fs_image(s, f)) implies
        finite_set_preimage_of_bijection(t, f).subset_eq(s)
} by {
    if is_bijection_fn(f) and t.subset_eq(fs_image(s, f)) {
        finite_set_preimage_subset_iff_of_bijection(t, fs_image(s, f), f)
        finite_set_preimage_of_bijection(t, f).subset_eq(
            finite_set_preimage_of_bijection(fs_image(s, f), f))
        finite_set_preimage_image_of_bijection(s, f)
        finite_set_preimage_of_bijection(fs_image(s, f), f) = s
        finite_set_preimage_of_bijection(t, f).subset_eq(s)
    }
}

/// Under a bijection, a finite-set image is a finite target exactly when the source is the
/// finite preimage of that target.
theorem finite_set_image_eq_iff_eq_preimage_of_bijection[T: Inhabited, U](s: FiniteSet[T],
    t: FiniteSet[U], f: T -> U) {
    is_bijection_fn(f) implies (fs_image(s, f) = t) =
        (s = finite_set_preimage_of_bijection(t, f))
} by {
    if is_bijection_fn(f) {
        if fs_image(s, f) = t {
            finite_set_preimage_image_of_bijection(s, f)
            finite_set_preimage_of_bijection(fs_image(s, f), f) = s
            finite_set_preimage_of_bijection(t, f) = s
            s = finite_set_preimage_of_bijection(t, f)
        }
        if s = finite_set_preimage_of_bijection(t, f) {
            finite_set_image_preimage_of_bijection(t, f)
            fs_image(finite_set_preimage_of_bijection(t, f), f) = t
            fs_image(s, f) = t
        }
        (fs_image(s, f) = t) = (s = finite_set_preimage_of_bijection(t, f))
    }
}
