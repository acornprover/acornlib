from finite_set.base import FiniteSet, finite_set_ext, finite_set_from_unique_list_cardinality_is_length,
    fs_from_list
from list import List, add_contains_left, add_contains_right, add_length,
    list_contains_implies_count_geq_one, map, not_contains_add,
    unique_implies_no_duplicate, unique_list_sum
from list import map_contains, map_contains_of_contains, map_length
from nat import Nat
from nat import sum_lte
from data.basic.set import Set, empty_set_contains_eq, insert_contains_eq, list_set,
    list_set_contains_eq, remove_contains_eq, set_ext, subset_contains,
    subset_contains_eq, subset_refl

lemma finite_set_from_list_nil_eq_empty[A] { FiniteSet.from_list[A](List.nil[A]) = FiniteSet.empty[A] } by {
    FiniteSet.from_list[A](List.nil[A]).underlying_set = list_set[A](List.nil[A])
    list_set[A](List.nil[A]) = Set.empty_set[A]
    FiniteSet.empty[A].underlying_set = Set.empty_set[A]
    finite_set_ext[A](FiniteSet.from_list[A](List.nil[A]), FiniteSet.empty[A])
}

/// Insert a fixed element into a finite subset.
/// This is the map used for the containing half of the subset enumerator.
define insert_into_subset[A](x: A, s: FiniteSet[A]) -> FiniteSet[A] {
    s.insert(x)
}

/// The list of finite subsets of the finite set represented by `items`.
/// Each recursive step keeps each old subset and also inserts the new head.
define list_subsets[A](items: List[A]) -> List[FiniteSet[A]] {
    match items {
        List.nil[A] {
            List.singleton[FiniteSet[A]](FiniteSet.empty[A])
        }
        List.cons[A](head, tail) {
            list_subsets[A](tail) + map[FiniteSet[A], FiniteSet[A]](
                list_subsets[A](tail),
                insert_into_subset[A](head)
            )
        }
    }
}


lemma singleton_contains_imp_eq[A](x: A, y: A) {
    List.singleton[A](x).contains(y) implies x = y
} by {
    if x != y {
        not List.singleton[A](x).contains(y)
        false
    }
}


lemma list_subsets_nil_contains_imp_eq_empty[A](t: FiniteSet[A]) {
    list_subsets[A](List.nil[A]).contains(t) implies t = FiniteSet.empty[A]
} by {
    singleton_contains_imp_eq[FiniteSet[A]](FiniteSet.empty[A], t)
}


lemma list_subsets_nil_sound[A](t: FiniteSet[A]) { list_subsets[A](List.nil[A]).contains(t) implies t.subset_eq(FiniteSet.from_list[A](List.nil[A])) } by {
    list_subsets_nil_contains_imp_eq_empty[A](t)
    finite_set_from_list_nil_eq_empty[A]
    subset_refl(t.underlying_set)
}


lemma finite_set_subset_empty_eq_empty[A](t: FiniteSet[A]) { t.subset_eq(FiniteSet.empty[A]) implies t = FiniteSet.empty[A] } by {
    t.subset_eq(FiniteSet.empty[A]) = t.underlying_set.subset(FiniteSet.empty[A].underlying_set)
    t.underlying_set.subset(FiniteSet.empty[A].underlying_set)
    forall(x: A) {
        if t.underlying_set.contains(x) {
            subset_contains(t.underlying_set, FiniteSet.empty[A].underlying_set, x)
            FiniteSet.empty[A].underlying_set = Set.empty_set[A]
            empty_set_contains_eq[A](x)
            false
        }
    }
    t.underlying_set.subset(Set.empty_set[A])
    subset_contains_eq(t.underlying_set, Set.empty_set[A])
    t.underlying_set = Set.empty_set[A]
    FiniteSet.empty[A].underlying_set = Set.empty_set[A]
    finite_set_ext[A](t, FiniteSet.empty[A])
}


lemma list_subsets_nil_contains_eq[A](t: FiniteSet[A]) { list_subsets[A](List.nil[A]).contains(t) = (t = FiniteSet.empty[A]) } by {
    list_subsets[A](List.nil[A]) = List.singleton[FiniteSet[A]](FiniteSet.empty[A])
    if list_subsets[A](List.nil[A]).contains(t) {
        List.singleton[FiniteSet[A]](FiniteSet.empty[A]).contains(t)
        singleton_contains_imp_eq[FiniteSet[A]](FiniteSet.empty[A], t)
    }
    if t = FiniteSet.empty[A] {
        list_subsets[A](List.nil[A]).contains(t)
    }
}


lemma list_subsets_nil_complete[A](t: FiniteSet[A]) { t.subset_eq(FiniteSet.from_list[A](List.nil[A])) implies list_subsets[A](List.nil[A]).contains(t) } by {
    finite_set_from_list_nil_eq_empty[A]
    finite_set_subset_empty_eq_empty[A](t)
    list_subsets_nil_contains_eq[A](t)
}


lemma finite_set_from_list_underlying[A](items: List[A]) {
    FiniteSet.from_list[A](items).underlying_set = list_set[A](items)
}


lemma tail_from_list_subset_cons_from_list[A](head: A, tail: List[A], t: FiniteSet[A]) { t.subset_eq(FiniteSet.from_list[A](tail)) implies t.subset_eq(FiniteSet.from_list[A](List.cons[A](head, tail))) } by {
    finite_set_from_list_underlying[A](tail)
    finite_set_from_list_underlying[A](List.cons[A](head, tail))
    t.subset_eq(FiniteSet.from_list[A](tail)) = t.underlying_set.subset(FiniteSet.from_list[A](tail).underlying_set)
    t.underlying_set.subset(FiniteSet.from_list[A](tail).underlying_set)
    t.underlying_set.subset(list_set[A](tail))
    forall(x: A) {
        if t.underlying_set.contains(x) {
            subset_contains(t.underlying_set, list_set[A](tail), x)
            list_set[A](tail).contains(x)
            list_set_contains_eq(tail, x)
            tail.contains(x)
            List.cons[A](head, tail).contains(x)
            list_set_contains_eq(List.cons[A](head, tail), x)
            list_set[A](List.cons[A](head, tail)).contains(x)
        }
    }
    subset_contains_eq(t.underlying_set, list_set[A](List.cons[A](head, tail)))
    t.underlying_set.subset(list_set[A](List.cons[A](head, tail)))
    t.underlying_set.subset(FiniteSet.from_list[A](List.cons[A](head, tail)).underlying_set)
    t.subset_eq(FiniteSet.from_list[A](List.cons[A](head, tail)))
}


lemma list_subsets_sound_cons_left[A](head: A, tail: List[A], t: FiniteSet[A]) { list_subsets[A](tail).contains(t) and t.subset_eq(FiniteSet.from_list[A](tail)) implies t.subset_eq(FiniteSet.from_list[A](List.cons[A](head, tail))) } by {
    tail_from_list_subset_cons_from_list[A](head, tail, t)
}


lemma insert_subset_cons_of_subset_tail[A](head: A, tail: List[A], t: FiniteSet[A]) {
    t.subset_eq(FiniteSet.from_list[A](tail)) implies t.insert(head).subset_eq(FiniteSet.from_list[A](List.cons[A](head, tail)))
} by {
    finite_set_from_list_underlying[A](tail)
    finite_set_from_list_underlying[A](List.cons[A](head, tail))
    t.insert(head).underlying_set = t.underlying_set.insert(head)
    t.subset_eq(FiniteSet.from_list[A](tail)) = t.underlying_set.subset(FiniteSet.from_list[A](tail).underlying_set)
    t.underlying_set.subset(FiniteSet.from_list[A](tail).underlying_set)
    t.underlying_set.subset(list_set[A](tail))
    forall(x: A) {
        if t.insert(head).underlying_set.contains(x) {
            t.underlying_set.insert(head).contains(x)
            insert_contains_eq(t.underlying_set, head, x)
            x = head or t.underlying_set.contains(x)
            if x = head {
                List.cons[A](head, tail).contains(x)
                list_set_contains_eq(List.cons[A](head, tail), x)
                list_set[A](List.cons[A](head, tail)).contains(x)
            }
            if t.underlying_set.contains(x) {
                subset_contains(t.underlying_set, list_set[A](tail), x)
                list_set[A](tail).contains(x)
                list_set_contains_eq(tail, x)
                tail.contains(x)
                List.cons[A](head, tail).contains(x)
                list_set_contains_eq(List.cons[A](head, tail), x)
                list_set[A](List.cons[A](head, tail)).contains(x)
            }
            list_set[A](List.cons[A](head, tail)).contains(x)
        }
    }
    subset_contains_eq(t.insert(head).underlying_set, list_set[A](List.cons[A](head, tail)))
    t.insert(head).underlying_set.subset(list_set[A](List.cons[A](head, tail)))
    t.insert(head).underlying_set.subset(FiniteSet.from_list[A](List.cons[A](head, tail)).underlying_set)
    t.insert(head).subset_eq(FiniteSet.from_list[A](List.cons[A](head, tail)))
}


lemma list_subsets_sound_cons_insert[A](head: A, tail: List[A], u: FiniteSet[A], t: FiniteSet[A]) {
    list_subsets[A](tail).contains(u) and u.subset_eq(FiniteSet.from_list[A](tail)) and u.insert(head) = t implies t.subset_eq(FiniteSet.from_list[A](List.cons[A](head, tail)))
} by {
    insert_subset_cons_of_subset_tail[A](head, tail, u)
    u.insert(head).subset_eq(FiniteSet.from_list[A](List.cons[A](head, tail)))
    t.subset_eq(FiniteSet.from_list[A](List.cons[A](head, tail)))
}


lemma list_subsets_cons_contains_cases[A](head: A, tail: List[A], t: FiniteSet[A]) { list_subsets[A](List.cons[A](head, tail)).contains(t) implies list_subsets[A](tail).contains(t) or exists(u: FiniteSet[A]) { list_subsets[A](tail).contains(u) and u.insert(head) = t } } by {
    list_subsets[A](List.cons[A](head, tail)) = list_subsets[A](tail) + map[FiniteSet[A], FiniteSet[A]](
        list_subsets[A](tail),
        insert_into_subset[A](head)
    )
    if list_subsets[A](tail).contains(t) {
        list_subsets[A](tail).contains(t) or exists(u: FiniteSet[A]) { list_subsets[A](tail).contains(u) and u.insert(head) = t }
    }
    if not list_subsets[A](tail).contains(t) {
        if not map[FiniteSet[A], FiniteSet[A]](list_subsets[A](tail), insert_into_subset[A](head)).contains(t) {
            not_contains_add[FiniteSet[A]](
                list_subsets[A](tail),
                map[FiniteSet[A], FiniteSet[A]](list_subsets[A](tail), insert_into_subset[A](head)),
                t
            )
            not (list_subsets[A](tail) + map[FiniteSet[A], FiniteSet[A]](list_subsets[A](tail), insert_into_subset[A](head))).contains(t)
            false
        }
        map_contains[FiniteSet[A], FiniteSet[A]](
            list_subsets[A](tail),
            insert_into_subset[A](head),
            t
        )
        let v: FiniteSet[A] satisfy { list_subsets[A](tail).contains(v) and insert_into_subset[A](head, v) = t }
        insert_into_subset[A](head, v) = v.insert(head)
        v.insert(head) = t
        exists(w: FiniteSet[A]) { list_subsets[A](tail).contains(w) and w.insert(head) = t }
        list_subsets[A](tail).contains(t) or exists(w: FiniteSet[A]) { list_subsets[A](tail).contains(w) and w.insert(head) = t }
    }
    list_subsets[A](tail).contains(t) or exists(w: FiniteSet[A]) { list_subsets[A](tail).contains(w) and w.insert(head) = t }
}


lemma list_subsets_sound_cons_from_cases[A](head: A, tail: List[A], t: FiniteSet[A]) { (forall(u: FiniteSet[A]) { list_subsets[A](tail).contains(u) implies u.subset_eq(FiniteSet.from_list[A](tail)) }) and list_subsets[A](List.cons[A](head, tail)).contains(t) implies t.subset_eq(FiniteSet.from_list[A](List.cons[A](head, tail))) } by {
    if not t.subset_eq(FiniteSet.from_list[A](List.cons[A](head, tail))) {
        list_subsets_cons_contains_cases[A](head, tail, t)
        if list_subsets[A](tail).contains(t) {
            t.subset_eq(FiniteSet.from_list[A](tail))
            list_subsets_sound_cons_left[A](head, tail, t)
            false
        }
        if not list_subsets[A](tail).contains(t) {
            exists(u: FiniteSet[A]) { list_subsets[A](tail).contains(u) and u.insert(head) = t }
            let u: FiniteSet[A] satisfy { list_subsets[A](tail).contains(u) and u.insert(head) = t }
            u.subset_eq(FiniteSet.from_list[A](tail))
            list_subsets_sound_cons_insert[A](head, tail, u, t)
            false
        }
    }
}


lemma list_subsets_sound[A](items: List[A], t: FiniteSet[A]) { list_subsets[A](items).contains(t) implies t.subset_eq(FiniteSet.from_list[A](items)) } by {

define p(xs: List[A]) -> Bool {
        forall(u: FiniteSet[A]) { list_subsets[A](xs).contains(u) implies u.subset_eq(FiniteSet.from_list[A](xs)) }
    }
    forall(u: FiniteSet[A]) {
        if list_subsets[A](List.nil[A]).contains(u) {
            list_subsets_nil_sound[A](u)
        }
    }
    p(List.nil[A])
    forall(head: A, tail: List[A]) {
        if p(tail) {
            forall(u: FiniteSet[A]) {
                if list_subsets[A](List.cons[A](head, tail)).contains(u) {
                    list_subsets_sound_cons_from_cases[A](head, tail, u)
                }
            }
            p(List.cons[A](head, tail))
        }
    }
    List.induction(function (xs: List[A]) {
        p(xs)
    })
    forall(xs: List[A]) { p(xs) }
    p(items)
}


lemma list_subsets_cons_contains_left[A](head: A, tail: List[A], t: FiniteSet[A]) { list_subsets[A](tail).contains(t) implies list_subsets[A](List.cons[A](head, tail)).contains(t) } by {
    list_subsets[A](List.cons[A](head, tail)) = list_subsets[A](tail) + map[FiniteSet[A], FiniteSet[A]](
        list_subsets[A](tail),
        insert_into_subset[A](head)
    )
    add_contains_left[FiniteSet[A]](
        list_subsets[A](tail),
        map[FiniteSet[A], FiniteSet[A]](list_subsets[A](tail), insert_into_subset[A](head)),
        t
    )
}


lemma list_subsets_cons_contains_insert[A](head: A, tail: List[A], t: FiniteSet[A]) { list_subsets[A](tail).contains(t) implies list_subsets[A](List.cons[A](head, tail)).contains(t.insert(head)) } by {
    map_contains_of_contains[FiniteSet[A], FiniteSet[A]](
        list_subsets[A](tail),
        insert_into_subset[A](head),
        t
    )
    insert_into_subset[A](head, t) = t.insert(head)
    map[FiniteSet[A], FiniteSet[A]](list_subsets[A](tail), insert_into_subset[A](head)).contains(t.insert(head))
    list_subsets[A](List.cons[A](head, tail)) = list_subsets[A](tail) + map[FiniteSet[A], FiniteSet[A]](
        list_subsets[A](tail),
        insert_into_subset[A](head)
    )
    add_contains_right[FiniteSet[A]](
        list_subsets[A](tail),
        map[FiniteSet[A], FiniteSet[A]](list_subsets[A](tail), insert_into_subset[A](head)),
        t.insert(head)
    )
}


lemma subset_cons_without_head_subset_tail[A](head: A, tail: List[A], t: FiniteSet[A]) { not t.contains(head) and t.subset_eq(FiniteSet.from_list[A](List.cons[A](head, tail))) implies t.subset_eq(FiniteSet.from_list[A](tail)) } by {
    finite_set_from_list_underlying[A](tail)
    finite_set_from_list_underlying[A](List.cons[A](head, tail))
    not t.contains(head)
    t.subset_eq(FiniteSet.from_list[A](List.cons[A](head, tail)))
    t.subset_eq(FiniteSet.from_list[A](List.cons[A](head, tail))) = t.underlying_set.subset(FiniteSet.from_list[A](List.cons[A](head, tail)).underlying_set)
    t.underlying_set.subset(FiniteSet.from_list[A](List.cons[A](head, tail)).underlying_set)
    t.underlying_set.subset(list_set[A](List.cons[A](head, tail)))
    forall(x: A) {
        if t.underlying_set.contains(x) {
            t.contains(x) = t.underlying_set.contains(x)
            t.contains(x)
            if x = head {
                t.contains(head)
                false
            }
            x != head
            subset_contains(t.underlying_set, list_set[A](List.cons[A](head, tail)), x)
            list_set[A](List.cons[A](head, tail)).contains(x)
            list_set_contains_eq(List.cons[A](head, tail), x)
            List.cons[A](head, tail).contains(x)
            tail.contains(x)
            list_set_contains_eq(tail, x)
            list_set[A](tail).contains(x)
        }
    }
    subset_contains_eq(t.underlying_set, list_set[A](tail))
    t.underlying_set.subset(list_set[A](tail))
    t.underlying_set.subset(FiniteSet.from_list[A](tail).underlying_set)
    t.subset_eq(FiniteSet.from_list[A](tail))
}


lemma remove_subset_tail_of_subset_cons_contains[A](head: A, tail: List[A], t: FiniteSet[A]) { t.contains(head) and t.subset_eq(FiniteSet.from_list[A](List.cons[A](head, tail))) implies t.remove(head).subset_eq(FiniteSet.from_list[A](tail)) } by {
    finite_set_from_list_underlying[A](tail)
    finite_set_from_list_underlying[A](List.cons[A](head, tail))
    t.subset_eq(FiniteSet.from_list[A](List.cons[A](head, tail)))
    t.subset_eq(FiniteSet.from_list[A](List.cons[A](head, tail))) = t.underlying_set.subset(FiniteSet.from_list[A](List.cons[A](head, tail)).underlying_set)
    t.underlying_set.subset(FiniteSet.from_list[A](List.cons[A](head, tail)).underlying_set)
    t.underlying_set.subset(list_set[A](List.cons[A](head, tail)))
    t.remove(head).underlying_set = t.underlying_set.remove(head)
    forall(x: A) {
        if t.remove(head).underlying_set.contains(x) {
            t.underlying_set.remove(head).contains(x)
            remove_contains_eq(t.underlying_set, head, x)
            t.underlying_set.contains(x)
            x != head
            subset_contains(t.underlying_set, list_set[A](List.cons[A](head, tail)), x)
            list_set[A](List.cons[A](head, tail)).contains(x)
            list_set_contains_eq(List.cons[A](head, tail), x)
            List.cons[A](head, tail).contains(x)
            tail.contains(x)
            list_set_contains_eq(tail, x)
            list_set[A](tail).contains(x)
        }
    }
    subset_contains_eq(t.remove(head).underlying_set, list_set[A](tail))
    t.remove(head).underlying_set.subset(list_set[A](tail))
    t.remove(head).underlying_set.subset(FiniteSet.from_list[A](tail).underlying_set)
    t.remove(head).subset_eq(FiniteSet.from_list[A](tail))
}


lemma insert_remove_eq_self_of_contains[A](t: FiniteSet[A], x: A) { t.contains(x) implies t.remove(x).insert(x) = t } by {
    t.remove(x).underlying_set = t.underlying_set.remove(x)
    t.remove(x).insert(x).underlying_set = t.remove(x).underlying_set.insert(x)
    forall(y: A) {
        if t.remove(x).insert(x).underlying_set.contains(y) {
            t.remove(x).underlying_set.insert(x).contains(y)
            insert_contains_eq(t.remove(x).underlying_set, x, y)
            y = x or t.remove(x).underlying_set.contains(y)
            if y = x {
                t.contains(y)
                t.underlying_set.contains(y)
            }
            if t.remove(x).underlying_set.contains(y) {
                t.underlying_set.remove(x).contains(y)
                remove_contains_eq(t.underlying_set, x, y)
                t.underlying_set.contains(y)
            }
            t.underlying_set.contains(y)
        }
        if t.underlying_set.contains(y) {
            if y = x {
                t.remove(x).underlying_set.insert(x).contains(y)
                t.remove(x).insert(x).underlying_set.contains(y)
            }
            if y != x {
                t.underlying_set.contains(y) and y != x
                remove_contains_eq(t.underlying_set, x, y)
                t.underlying_set.remove(x).contains(y)
                t.remove(x).underlying_set.contains(y)
                insert_contains_eq(t.remove(x).underlying_set, x, y)
                t.remove(x).underlying_set.insert(x).contains(y)
                t.remove(x).insert(x).underlying_set.contains(y)
            }
            t.remove(x).insert(x).underlying_set.contains(y)
        }
        t.remove(x).insert(x).underlying_set.contains(y) = t.underlying_set.contains(y)
    }
    t.remove(x).insert(x).underlying_set = t.underlying_set
    finite_set_ext[A](t.remove(x).insert(x), t)
}


lemma list_subsets_complete_cons_step[A](head: A, tail: List[A], t: FiniteSet[A]) {
    (forall(u: FiniteSet[A]) { u.subset_eq(FiniteSet.from_list[A](tail)) implies list_subsets[A](tail).contains(u) }) and t.subset_eq(FiniteSet.from_list[A](List.cons[A](head, tail))) implies list_subsets[A](List.cons[A](head, tail)).contains(t)
} by {
    if t.contains(head) {
        remove_subset_tail_of_subset_cons_contains[A](head, tail, t)
        t.remove(head).subset_eq(FiniteSet.from_list[A](tail))
        list_subsets[A](tail).contains(t.remove(head))
        list_subsets_cons_contains_insert[A](head, tail, t.remove(head))
        list_subsets[A](List.cons[A](head, tail)).contains(t.remove(head).insert(head))
        insert_remove_eq_self_of_contains[A](t, head)
        t.remove(head).insert(head) = t
        list_subsets[A](List.cons[A](head, tail)).contains(t)
    }
    if not t.contains(head) {
        subset_cons_without_head_subset_tail[A](head, tail, t)
        t.subset_eq(FiniteSet.from_list[A](tail))
        list_subsets[A](tail).contains(t)
        list_subsets_cons_contains_left[A](head, tail, t)
        list_subsets[A](List.cons[A](head, tail)).contains(t)
    }
}

lemma list_subsets_complete[A](items: List[A], t: FiniteSet[A]) { t.subset_eq(FiniteSet.from_list[A](items)) implies list_subsets[A](items).contains(t) } by {
    define p(xs: List[A]) -> Bool {
        forall(u: FiniteSet[A]) { u.subset_eq(FiniteSet.from_list[A](xs)) implies list_subsets[A](xs).contains(u) }
    }
    forall(u: FiniteSet[A]) {
        if u.subset_eq(FiniteSet.from_list[A](List.nil[A])) {
            list_subsets_nil_complete[A](u)
        }
    }
    p(List.nil[A])
    forall(head: A, tail: List[A]) {
        if p(tail) {
            forall(u: FiniteSet[A]) {
                if u.subset_eq(FiniteSet.from_list[A](List.cons[A](head, tail))) {
                    list_subsets_complete_cons_step[A](head, tail, u)
                }
            }
            p(List.cons[A](head, tail))
        }
    }
    List.induction(function (xs: List[A]) { p(xs) })
    forall(xs: List[A]) { p(xs) }
    p(items)
    t.subset_eq(FiniteSet.from_list[A](items))
    list_subsets[A](items).contains(t)
}

/// A finite set occurs in `list_subsets(items)` exactly when it is a subset of `FiniteSet.from_list(items)`.
theorem list_subsets_exact[A](items: List[A], t: FiniteSet[A]) { list_subsets[A](items).contains(t) = t.subset_eq(FiniteSet.from_list[A](items)) } by {
    if list_subsets[A](items).contains(t) {
        list_subsets_sound[A](items, t)
    }
    if t.subset_eq(FiniteSet.from_list[A](items)) {
        list_subsets_complete[A](items, t)
    }
    list_subsets[A](items).contains(t) = t.subset_eq(FiniteSet.from_list[A](items))
}


lemma insert_into_subset_injective[A](x: A, s: FiniteSet[A], t: FiniteSet[A]) { s.insert(x) = t.insert(x) implies s.remove(x) = t.remove(x) } by {
    s.insert(x).underlying_set = t.insert(x).underlying_set
    s.insert(x).underlying_set = s.underlying_set.insert(x)
    t.insert(x).underlying_set = t.underlying_set.insert(x)
    s.underlying_set.insert(x) = t.underlying_set.insert(x)
    s.remove(x).underlying_set = s.underlying_set.remove(x)
    t.remove(x).underlying_set = t.underlying_set.remove(x)
    forall(y: A) {
        if s.remove(x).underlying_set.contains(y) {
            s.underlying_set.remove(x).contains(y)
            remove_contains_eq(s.underlying_set, x, y)
            s.underlying_set.contains(y) and y != x
            s.underlying_set.contains(y)
            y != x
            insert_contains_eq(s.underlying_set, x, y)
            s.underlying_set.insert(x).contains(y)
            t.underlying_set.insert(x).contains(y)
            insert_contains_eq(t.underlying_set, x, y)
            y = x or t.underlying_set.contains(y)
            if y = x {
                false
            }
            t.underlying_set.contains(y)
            t.underlying_set.contains(y) and y != x
            remove_contains_eq(t.underlying_set, x, y)
            t.underlying_set.remove(x).contains(y)
            t.remove(x).underlying_set.contains(y)
        }
        if t.remove(x).underlying_set.contains(y) {
            t.underlying_set.remove(x).contains(y)
            remove_contains_eq(t.underlying_set, x, y)
            t.underlying_set.contains(y) and y != x
            t.underlying_set.contains(y)
            y != x
            insert_contains_eq(t.underlying_set, x, y)
            t.underlying_set.insert(x).contains(y)
            s.underlying_set.insert(x).contains(y)
            insert_contains_eq(s.underlying_set, x, y)
            y = x or s.underlying_set.contains(y)
            if y = x {
                false
            }
            s.underlying_set.contains(y)
            s.underlying_set.contains(y) and y != x
            remove_contains_eq(s.underlying_set, x, y)
            s.underlying_set.remove(x).contains(y)
            s.remove(x).underlying_set.contains(y)
        }
        s.remove(x).underlying_set.contains(y) = t.remove(x).underlying_set.contains(y)
    }
    s.remove(x).underlying_set = t.remove(x).underlying_set
    finite_set_ext[A](s.remove(x), t.remove(x))
}


lemma subset_of_from_list_not_contains_head[A](head: A, tail: List[A], t: FiniteSet[A]) { not tail.contains(head) and t.subset_eq(FiniteSet.from_list[A](tail)) implies not t.contains(head) } by {
    finite_set_from_list_underlying[A](tail)
    t.subset_eq(FiniteSet.from_list[A](tail)) = t.underlying_set.subset(FiniteSet.from_list[A](tail).underlying_set)
    t.underlying_set.subset(FiniteSet.from_list[A](tail).underlying_set)
    t.underlying_set.subset(list_set[A](tail))
    if t.contains(head) {
        t.contains(head) = t.underlying_set.contains(head)
        t.underlying_set.contains(head)
        subset_contains(t.underlying_set, list_set[A](tail), head)
        list_set[A](tail).contains(head)
        list_set_contains_eq(tail, head)
        tail.contains(head)
        false
    }
}

lemma list_subsets_tail_members_avoid_head[A](head: A, tail: List[A], t: FiniteSet[A]) { not tail.contains(head) and list_subsets[A](tail).contains(t) implies not t.contains(head) } by {
    list_subsets_sound[A](tail, t)
    subset_of_from_list_not_contains_head[A](head, tail, t)
}

lemma pow2_suc(n: Nat) { Nat.2.pow(n.suc) = Nat.2.pow(n) + Nat.2.pow(n) } by {
    Nat.2.pow(n.suc) = Nat.2.pow(n) * Nat.2
    Nat.2.pow(n) * Nat.2 = Nat.2.pow(n) * (Nat.1 + Nat.1)
    Nat.2.pow(n) * (Nat.1 + Nat.1) = Nat.2.pow(n) * Nat.1 + Nat.2.pow(n) * Nat.1
    Nat.2.pow(n) * Nat.1 = Nat.2.pow(n)
    Nat.2.pow(n.suc) = Nat.2.pow(n) + Nat.2.pow(n)
}

lemma list_subsets_nil_length[A] {
    list_subsets[A](List.nil[A]).length = Nat.2.pow(List.nil[A].length)
} by {
    list_subsets[A](List.nil[A]) = List.singleton[FiniteSet[A]](FiniteSet.empty[A])
    List.singleton[FiniteSet[A]](FiniteSet.empty[A]) =
        List.cons[FiniteSet[A]](FiniteSet.empty[A], List.nil[FiniteSet[A]])
    List.nil[FiniteSet[A]].length = Nat.0
    List.cons[FiniteSet[A]](FiniteSet.empty[A], List.nil[FiniteSet[A]]).length = Nat.1
    list_subsets[A](List.nil[A]).length = Nat.1
    List.nil[A].length = Nat.0
    Nat.2.pow(List.nil[A].length) = Nat.1
    list_subsets[A](List.nil[A]).length = Nat.2.pow(List.nil[A].length)
}

lemma list_subsets_length_cons[A](head: A, tail: List[A]) {
    list_subsets[A](List.cons[A](head, tail)).length =
        list_subsets[A](tail).length + list_subsets[A](tail).length
} by {
    list_subsets[A](List.cons[A](head, tail)) =
        list_subsets[A](tail) + map[FiniteSet[A], FiniteSet[A]](
            list_subsets[A](tail),
            insert_into_subset[A](head)
        )
    add_length[FiniteSet[A]](
        list_subsets[A](tail),
        map[FiniteSet[A], FiniteSet[A]](list_subsets[A](tail), insert_into_subset[A](head))
    )
    map_length[FiniteSet[A], FiniteSet[A]](list_subsets[A](tail), insert_into_subset[A](head))
    map[FiniteSet[A], FiniteSet[A]](list_subsets[A](tail), insert_into_subset[A](head)).length =
        list_subsets[A](tail).length
}

lemma list_subsets_cons_length[A](head: A, tail: List[A]) {
    list_subsets[A](tail).length = Nat.2.pow(tail.length) implies
        list_subsets[A](List.cons[A](head, tail)).length =
        Nat.2.pow(List.cons[A](head, tail).length)
} by {
    list_subsets_length_cons[A](head, tail)
    list_subsets[A](List.cons[A](head, tail)).length =
        list_subsets[A](tail).length + list_subsets[A](tail).length
    list_subsets[A](tail).length = Nat.2.pow(tail.length)
    list_subsets[A](List.cons[A](head, tail)).length =
        Nat.2.pow(tail.length) + Nat.2.pow(tail.length)
    pow2_suc(tail.length)
    Nat.2.pow(tail.length.suc) = Nat.2.pow(tail.length) + Nat.2.pow(tail.length)
    list_subsets[A](List.cons[A](head, tail)).length = Nat.2.pow(tail.length.suc)
    List.cons[A](head, tail).length = tail.length.suc
    list_subsets[A](List.cons[A](head, tail)).length =
        Nat.2.pow(List.cons[A](head, tail).length)
}

/// The recursive subset enumerator has length `2 ^ items.length`.
theorem list_subsets_length[A](items: List[A]) {
    list_subsets[A](items).length = Nat.2.pow(items.length)
} by {
    define p(xs: List[A]) -> Bool {
        list_subsets[A](xs).length = Nat.2.pow(xs.length)
    }

    list_subsets_nil_length[A]
    p(List.nil[A])

    forall(head: A, tail: List[A]) {
        if p(tail) {
            list_subsets[A](tail).length = Nat.2.pow(tail.length)
            list_subsets_cons_length[A](head, tail)
            list_subsets[A](List.cons[A](head, tail)).length = Nat.2.pow(List.cons[A](head, tail).length)
            p(List.cons[A](head, tail))
        }
    }

    List.induction(function(xs: List[A]) { p(xs) })
    forall(xs: List[A]) { p(xs) }
    p(items)
    list_subsets[A](items).length = Nat.2.pow(items.length)
}

lemma finite_set_remove_underlying_set[A](s: FiniteSet[A], x: A) {
    s.remove(x).underlying_set = s.underlying_set.remove(x)
}

lemma set_remove_of_not_contains_membership[A](s: Set[A], x: A, y: A) {
    not s.contains(x) implies s.remove(x).contains(y) = s.contains(y)
} by {
    s.remove(x).contains(y) = (s.contains(y) and y != x)
    if y = x {
        not s.contains(y)
        s.remove(x).contains(y) = false
        s.contains(y) = false
    }
    if y != x {
        (s.contains(y) and y != x) = s.contains(y)
        s.remove(x).contains(y) = s.contains(y)
    }
}

lemma set_remove_of_not_contains[A](s: Set[A], x: A) {
    not s.contains(x) implies s.remove(x) = s
} by {
    forall(y: A) {
        set_remove_of_not_contains_membership[A](s, x, y)
        s.remove(x).contains(y) = s.contains(y)
    }
    set_ext[A](s.remove(x), s)
}

lemma finite_set_remove_of_not_contains[A](s: FiniteSet[A], x: A) {
    not s.contains(x) implies s.remove(x) = s
} by {
    not s.underlying_set.contains(x)
    set_remove_of_not_contains[A](s.underlying_set, x)
    s.underlying_set.remove(x) = s.underlying_set
    finite_set_remove_underlying_set[A](s, x)
    s.remove(x).underlying_set = s.underlying_set
    finite_set_ext[A](s.remove(x), s)
}

lemma cons_unique_not_tail_contains_by_count[A](head: A, tail: List[A]) {
    List.cons[A](head, tail).is_unique implies not tail.contains(head)
} by {
    if tail.contains(head) {
        list_contains_implies_count_geq_one[A](tail, head)
        tail.count(head) >= Nat.1
        List.cons(head, tail).count(head) = Nat.1 + tail.count(head)
        sum_lte(Nat.1, Nat.1, Nat.1, tail.count(head))
        Nat.1 + Nat.1 <= Nat.1 + tail.count(head)
        Nat.1 + Nat.1 = Nat.1.suc
        Nat.1.suc <= Nat.1 + tail.count(head)
        List.cons(head, tail).count(head) >= Nat.1.suc
        List.cons(head, tail).count(head) > Nat.1
        unique_implies_no_duplicate[A](List.cons(head, tail), head)
        List.cons(head, tail).count(head) <= Nat.1
        false
    }
}

lemma mapped_insert_contains_reflect_on_avoid[A](head: A, s: FiniteSet[A], rest: List[FiniteSet[A]]) {
    not s.contains(head) and
    forall(u: FiniteSet[A]) { rest.contains(u) implies not u.contains(head) } and
    map[FiniteSet[A], FiniteSet[A]](rest, function (t: FiniteSet[A]) { insert_into_subset[A](head, t) }).contains(insert_into_subset[A](head, s))
    implies rest.contains(s)
} by {
    let u: FiniteSet[A] satisfy {
        rest.contains(u) and
        insert_into_subset[A](head, u) = insert_into_subset[A](head, s)
    }
    not u.contains(head)
    insert_into_subset[A](head, u) = u.insert(head)
    insert_into_subset[A](head, s) = s.insert(head)
    u.insert(head) = s.insert(head)
    u.remove(head) = s.remove(head)
    u.remove(head) = u
    s.remove(head) = s
    u = s
    rest.contains(s)
}

lemma list_cons_unique_intro_generic[B](x: B, tail: List[B]) {
    not tail.contains(x) and tail.is_unique implies List.cons[B](x, tail).is_unique
} by {
    tail.unique = tail
    List.cons[B](x, tail).unique = List.cons[B](x, tail.unique)
    List.cons[B](x, tail).unique = List.cons[B](x, tail)
}

lemma map_insert_into_subset_cons_eq[A](head: A, s: FiniteSet[A], rest: List[FiniteSet[A]]) {
    map[FiniteSet[A], FiniteSet[A]](List.cons[FiniteSet[A]](s, rest), function (t: FiniteSet[A]) { insert_into_subset[A](head, t) }) =
    List.cons[FiniteSet[A]](insert_into_subset[A](head, s), map[FiniteSet[A], FiniteSet[A]](rest, function (t: FiniteSet[A]) { insert_into_subset[A](head, t) }))
}

lemma mapped_insert_cons_unique_from_tail[A](head: A, s: FiniteSet[A], rest: List[FiniteSet[A]]) { not map[FiniteSet[A], FiniteSet[A]](rest, function (t: FiniteSet[A]) { insert_into_subset[A](head, t) }).contains(insert_into_subset[A](head, s)) and map[FiniteSet[A], FiniteSet[A]](rest, function (t: FiniteSet[A]) { insert_into_subset[A](head, t) }).is_unique implies map[FiniteSet[A], FiniteSet[A]](List.cons[FiniteSet[A]](s, rest), function (t: FiniteSet[A]) { insert_into_subset[A](head, t) }).is_unique } by {
    List.cons[FiniteSet[A]](insert_into_subset[A](head, s), map[FiniteSet[A], FiniteSet[A]](rest, function (t: FiniteSet[A]) { insert_into_subset[A](head, t) })).is_unique
    map[FiniteSet[A], FiniteSet[A]](List.cons[FiniteSet[A]](s, rest), function (t: FiniteSet[A]) { insert_into_subset[A](head, t) }) = List.cons[FiniteSet[A]](insert_into_subset[A](head, s), map[FiniteSet[A], FiniteSet[A]](rest, function (t: FiniteSet[A]) { insert_into_subset[A](head, t) }))
}

lemma map_insert_into_subset_nil_unique[A](head: A) { map[FiniteSet[A], FiniteSet[A]](List.nil[FiniteSet[A]], function (s: FiniteSet[A]) { insert_into_subset[A](head, s) }).is_unique } by {
    map[FiniteSet[A], FiniteSet[A]](List.nil[FiniteSet[A]], function (s: FiniteSet[A]) { insert_into_subset[A](head, s) }) = List.nil[FiniteSet[A]]
    List.nil[FiniteSet[A]].unique = List.nil[FiniteSet[A]]
    List.nil[FiniteSet[A]].is_unique
}

lemma cons_tail_avoid_head[A](head: A, s: FiniteSet[A], rest: List[FiniteSet[A]]) { forall(u: FiniteSet[A]) { List.cons[FiniteSet[A]](s, rest).contains(u) implies not u.contains(head) } implies forall(u: FiniteSet[A]) { rest.contains(u) implies not u.contains(head) } } by {
    forall(u: FiniteSet[A]) {
        if rest.contains(u) {
            List.cons[FiniteSet[A]](s, rest).contains(u)
            not u.contains(head)
        }
    }
}

lemma mapped_insert_tail_excludes_head_image[A](head: A, s: FiniteSet[A], rest: List[FiniteSet[A]]) { not s.contains(head) and forall(u: FiniteSet[A]) { rest.contains(u) implies not u.contains(head) } and List.cons[FiniteSet[A]](s, rest).is_unique implies not map[FiniteSet[A], FiniteSet[A]](rest, function (t: FiniteSet[A]) { insert_into_subset[A](head, t) }).contains(insert_into_subset[A](head, s)) } by {
    if map[FiniteSet[A], FiniteSet[A]](rest, function (t: FiniteSet[A]) { insert_into_subset[A](head, t) }).contains(insert_into_subset[A](head, s)) {
        rest.contains(s)
        not rest.contains(s)
        false
    }
}

lemma map_insert_into_subset_cons_unique_from_avoid_and_tail[A](head: A, s: FiniteSet[A], rest: List[FiniteSet[A]]) { List.cons[FiniteSet[A]](s, rest).is_unique and forall(u: FiniteSet[A]) { List.cons[FiniteSet[A]](s, rest).contains(u) implies not u.contains(head) } and map[FiniteSet[A], FiniteSet[A]](rest, function (t: FiniteSet[A]) { insert_into_subset[A](head, t) }).is_unique implies map[FiniteSet[A], FiniteSet[A]](List.cons[FiniteSet[A]](s, rest), function (t: FiniteSet[A]) { insert_into_subset[A](head, t) }).is_unique } by {
    List.cons[FiniteSet[A]](s, rest).contains(s)
    not s.contains(head)
    forall(u: FiniteSet[A]) {
        if rest.contains(u) {
            List.cons[FiniteSet[A]](s, rest).contains(u)
            not u.contains(head)
        }
    }
    not map[FiniteSet[A], FiniteSet[A]](rest, function (t: FiniteSet[A]) { insert_into_subset[A](head, t) }).contains(insert_into_subset[A](head, s))
    map[FiniteSet[A], FiniteSet[A]](List.cons[FiniteSet[A]](s, rest), function (t: FiniteSet[A]) { insert_into_subset[A](head, t) }).is_unique
}

lemma map_insert_into_subset_cons_unique_from_ih[A](head: A, s: FiniteSet[A], rest: List[FiniteSet[A]]) { (rest.is_unique and forall(u: FiniteSet[A]) { rest.contains(u) implies not u.contains(head) } implies map[FiniteSet[A], FiniteSet[A]](rest, function (t: FiniteSet[A]) { insert_into_subset[A](head, t) }).is_unique) and List.cons[FiniteSet[A]](s, rest).is_unique and forall(u: FiniteSet[A]) { List.cons[FiniteSet[A]](s, rest).contains(u) implies not u.contains(head) } implies map[FiniteSet[A], FiniteSet[A]](List.cons[FiniteSet[A]](s, rest), function (t: FiniteSet[A]) { insert_into_subset[A](head, t) }).is_unique } by {
    rest.is_unique
    forall(u: FiniteSet[A]) {
        if rest.contains(u) {
            List.cons[FiniteSet[A]](s, rest).contains(u)
            not u.contains(head)
        }
    }
    map[FiniteSet[A], FiniteSet[A]](rest, function (t: FiniteSet[A]) { insert_into_subset[A](head, t) }).is_unique
    map_insert_into_subset_cons_unique_from_avoid_and_tail[A](head, s, rest)
}

lemma locally_injective_map_is_unique[T, U](items: List[T], f: T -> U) {
    items.is_unique and forall(x: T, y: T) { items.contains(x) and items.contains(y) and f(x) = f(y) implies x = y }
    implies map[T, U](items, f).is_unique
} by {
    define p(l: List[T]) -> Bool {
        l.is_unique and forall(x: T, y: T) { l.contains(x) and l.contains(y) and f(x) = f(y) implies x = y }
        implies map[T, U](l, f).is_unique
    }

    map[T, U](List.nil[T], f) = List.nil[U]
    List.nil[U].unique = List.nil[U]
    List.nil[U].is_unique
    p(List.nil[T])

    forall(head: T, tail: List[T]) {
        if p(tail) {
            if List.cons[T](head, tail).is_unique and forall(x: T, y: T) { List.cons[T](head, tail).contains(x) and List.cons[T](head, tail).contains(y) and f(x) = f(y) implies x = y } {
                tail.is_unique
                forall(x: T, y: T) {
                    if tail.contains(x) and tail.contains(y) and f(x) = f(y) {
                        List.cons[T](head, tail).contains(x)
                        List.cons[T](head, tail).contains(y)
                        x = y
                    }
                }
                tail.is_unique and forall(x: T, y: T) { tail.contains(x) and tail.contains(y) and f(x) = f(y) implies x = y }
                p(tail)
                map[T, U](tail, f).is_unique
                if map[T, U](tail, f).contains(f(head)) {
                    map_contains[T, U](tail, f, f(head))
                    let x: T satisfy {
                        tail.contains(x) and f(x) = f(head)
                    }
                    List.cons[T](head, tail).contains(x)
                    List.cons[T](head, tail).contains(head)
                    x = head
                    tail.contains(head)
                    cons_unique_not_tail_contains_by_count[T](head, tail)
                    false
                }
                not map[T, U](tail, f).contains(f(head))
                list_cons_unique_intro_generic[U](f(head), map[T, U](tail, f))
                List.cons[U](f(head), map[T, U](tail, f)).is_unique
                map[T, U](List.cons[T](head, tail), f) = List.cons[U](f(head), map[T, U](tail, f))
                map[T, U](List.cons[T](head, tail), f).is_unique
            }
            p(List.cons[T](head, tail))
        }
    }

    List.induction(function(l: List[T]) { p(l) })
    p(items)
}

lemma map_insert_into_subset_unique_on_avoid_head[A](head: A, xs: List[FiniteSet[A]]) {
    xs.is_unique and forall(s: FiniteSet[A]) { xs.contains(s) implies not s.contains(head) } implies map[FiniteSet[A], FiniteSet[A]](xs, function (s: FiniteSet[A]) { insert_into_subset[A](head, s) }).is_unique
} by {
    forall(s: FiniteSet[A], t: FiniteSet[A]) {
        if xs.contains(s) and xs.contains(t) and (function (u: FiniteSet[A]) { insert_into_subset[A](head, u) })(s) = (function (u: FiniteSet[A]) { insert_into_subset[A](head, u) })(t) {
            not s.contains(head)
            not t.contains(head)
            insert_into_subset[A](head, s) = insert_into_subset[A](head, t)
            s.insert(head) = t.insert(head)
            s.remove(head) = t.remove(head)
            s.remove(head) = s
            t.remove(head) = t
            s = t
        }
    }
    locally_injective_map_is_unique[FiniteSet[A], FiniteSet[A]](xs, function (s: FiniteSet[A]) { insert_into_subset[A](head, s) })
}

lemma insert_into_subset_contains_head[A](head: A, t: FiniteSet[A]) {
    insert_into_subset[A](head, t).contains(head)
} by {
    insert_into_subset[A](head, t) = t.insert(head)
    insert_into_subset[A](head, t).underlying_set = t.underlying_set.insert(head)
    insert_contains_eq(t.underlying_set, head, head)
}

lemma list_subsets_mapped_members_contain_head[A](head: A, tail: List[A], t: FiniteSet[A]) {
    map[FiniteSet[A], FiniteSet[A]](list_subsets[A](tail), insert_into_subset[A](head)).contains(t)
        implies t.contains(head)
} by {
    map_contains(list_subsets[A](tail), insert_into_subset[A](head), t)
    let u: FiniteSet[A] satisfy {
        list_subsets[A](tail).contains(u) and insert_into_subset[A](head, u) = t
    }
    insert_into_subset_contains_head[A](head, u)
    t.contains(head)
}

lemma list_subsets_halves_disjoint[A](head: A, tail: List[A], t: FiniteSet[A]) {
    not tail.contains(head) and list_subsets[A](tail).contains(t) and
        map[FiniteSet[A], FiniteSet[A]](list_subsets[A](tail), insert_into_subset[A](head)).contains(t)
        implies false
} by {
    list_subsets_tail_members_avoid_head[A](head, tail, t)
    list_subsets_mapped_members_contain_head[A](head, tail, t)
    false
}

lemma list_subsets_halves_disjoint_from_cons_unique[A](head: A, tail: List[A]) {
    List.cons[A](head, tail).is_unique implies forall(t: FiniteSet[A]) {
        not (
            list_subsets[A](tail).contains(t) and
            map[FiniteSet[A], FiniteSet[A]](list_subsets[A](tail), insert_into_subset[A](head)).contains(t)
        )
    }
} by {
    cons_unique_not_tail_contains_by_count[A](head, tail)
    forall(t: FiniteSet[A]) {
        if list_subsets[A](tail).contains(t) and
            map[FiniteSet[A], FiniteSet[A]](list_subsets[A](tail), insert_into_subset[A](head)).contains(t) {
            list_subsets_halves_disjoint[A](head, tail, t)
            false
        }
    }
}

lemma list_subsets_mapped_insert_unique_from_cons_unique[A](head: A, tail: List[A]) {
    (tail.is_unique implies list_subsets[A](tail).is_unique) and
        List.cons[A](head, tail).is_unique implies
        map[FiniteSet[A], FiniteSet[A]](list_subsets[A](tail), insert_into_subset[A](head)).is_unique
} by {
    cons_unique_not_tail_contains_by_count[A](head, tail)
    tail.is_unique
    list_subsets[A](tail).is_unique
    forall(s: FiniteSet[A]) {
        if list_subsets[A](tail).contains(s) {
            list_subsets_tail_members_avoid_head[A](head, tail, s)
            not s.contains(head)
        }
    }
    map_insert_into_subset_unique_on_avoid_head[A](head, list_subsets[A](tail))
}

lemma list_subsets_unique_cons_step_sum_bridge[A](head: A, tail: List[A]) { (tail.is_unique implies list_subsets[A](tail).is_unique) and List.cons[A](head, tail).is_unique implies (list_subsets[A](tail) + map[FiniteSet[A], FiniteSet[A]](list_subsets[A](tail), insert_into_subset[A](head))).is_unique } by {
    tail.is_unique
    list_subsets[A](tail).is_unique
    list_subsets_mapped_insert_unique_from_cons_unique[A](head, tail)
    map[FiniteSet[A], FiniteSet[A]](list_subsets[A](tail), insert_into_subset[A](head)).is_unique
    list_subsets_halves_disjoint_from_cons_unique[A](head, tail)
    forall(t: FiniteSet[A]) { not (list_subsets[A](tail).contains(t) and map[FiniteSet[A], FiniteSet[A]](list_subsets[A](tail), insert_into_subset[A](head)).contains(t)) }
    unique_list_sum[FiniteSet[A]](list_subsets[A](tail), map[FiniteSet[A], FiniteSet[A]](list_subsets[A](tail), insert_into_subset[A](head)))
}

lemma list_subsets_cons_unique_from_sum_bridge[A](head: A, tail: List[A]) { (list_subsets[A](tail) + map[FiniteSet[A], FiniteSet[A]](list_subsets[A](tail), insert_into_subset[A](head))).is_unique implies list_subsets[A](List.cons[A](head, tail)).is_unique } by {
    list_subsets[A](List.cons[A](head, tail)) = list_subsets[A](tail) + map[FiniteSet[A], FiniteSet[A]](list_subsets[A](tail), insert_into_subset[A](head))
}

lemma list_subsets_unique_cons_step[A](head: A, tail: List[A]) { (tail.is_unique implies list_subsets[A](tail).is_unique) and List.cons[A](head, tail).is_unique implies list_subsets[A](List.cons[A](head, tail)).is_unique } by {
    list_subsets_unique_cons_step_sum_bridge[A](head, tail)
    (list_subsets[A](tail) + map[FiniteSet[A], FiniteSet[A]](list_subsets[A](tail), insert_into_subset[A](head))).is_unique
    list_subsets_cons_unique_from_sum_bridge[A](head, tail)
}

/// If the input list has no repeated elements, the subset enumerator has no repeated finite subsets.
theorem list_subsets_unique[A](items: List[A]) {
    items.is_unique implies list_subsets[A](items).is_unique
} by {
    define q(xs: List[A]) -> Bool {
        xs.is_unique implies list_subsets[A](xs).is_unique
    }
    q(List.nil[A])
    forall(head: A, tail: List[A]) {
        if q(tail) {
            list_subsets_unique_cons_step[A](head, tail)
            q(List.cons[A](head, tail))
        }
    }
    List.induction(function(xs: List[A]) { q(xs) })
    forall(xs: List[A]) { q(xs) }
    q(items)
}

lemma finite_set_from_list_contains_eq[B](items: List[B], x: B) {
    FiniteSet.from_list[B](items).contains(x) = items.contains(x)
} by {
    FiniteSet.from_list[B](items).underlying_set = list_set[B](items)
    FiniteSet.from_list[B](items).contains(x) = FiniteSet.from_list[B](items).underlying_set.contains(x)
    FiniteSet.from_list[B](items).underlying_set.contains(x) = list_set[B](items).contains(x)
    list_set[B](items).contains(x) = items.contains(x)
}

lemma fs_from_list_contains_eq[B](items: List[B], x: B) {
    fs_from_list[B](items).contains(x) = items.contains(x)
} by {
    fs_from_list[B](items).underlying_set = list_set[B](items)
    fs_from_list[B](items).contains(x) = fs_from_list[B](items).underlying_set.contains(x)
    fs_from_list[B](items).underlying_set.contains(x) = list_set[B](items).contains(x)
    list_set[B](items).contains(x) = items.contains(x)
}

/// The finite powerset represented by the subset list associated to `items`.
define finite_powerset_from_list[A](items: List[A]) -> FiniteSet[FiniteSet[A]] {
    fs_from_list[FiniteSet[A]](list_subsets[A](items))
}

lemma finite_powerset_from_list_contains_eq[A](items: List[A], s: FiniteSet[A]) {
    finite_powerset_from_list[A](items).contains(s) = list_subsets[A](items).contains(s)
} by {
    finite_powerset_from_list[A](items) = fs_from_list[FiniteSet[A]](list_subsets[A](items))
    fs_from_list[FiniteSet[A]](list_subsets[A](items)).contains(s) = list_subsets[A](items).contains(s)
}

/// The list-backed finite powerset contains exactly the finite subsets of `FiniteSet.from_list(items)`.
theorem finite_powerset_from_list_exact[A](items: List[A]) {
    forall(s: FiniteSet[A]) {
        finite_powerset_from_list[A](items).contains(s) = s.subset_eq(FiniteSet.from_list[A](items))
    }
} by {
    forall(s: FiniteSet[A]) {
        finite_powerset_from_list_contains_eq[A](items, s)
        finite_powerset_from_list[A](items).contains(s) = list_subsets[A](items).contains(s)
        list_subsets[A](items).contains(s) = s.subset_eq(FiniteSet.from_list[A](items))
        finite_powerset_from_list[A](items).contains(s) = s.subset_eq(FiniteSet.from_list[A](items))
    }
}

/// A list-backed finite powerset over a duplicate-free base has cardinality `2 ^ items.length`.
theorem finite_powerset_from_list_cardinality[A](items: List[A]) {
    items.is_unique implies finite_powerset_from_list[A](items).cardinality_is(Nat.2.pow(items.length))
} by {
    list_subsets[A](items).is_unique
    finite_powerset_from_list[A](items) = fs_from_list[FiniteSet[A]](list_subsets[A](items))
    finite_set_from_unique_list_cardinality_is_length[FiniteSet[A]](list_subsets[A](items))
    fs_from_list[FiniteSet[A]](list_subsets[A](items)).cardinality_is(list_subsets[A](items).length)
    finite_powerset_from_list[A](items).cardinality_is(list_subsets[A](items).length)
    list_subsets[A](items).length = Nat.2.pow(items.length)
    finite_powerset_from_list[A](items).cardinality_is(Nat.2.pow(items.length))
}
