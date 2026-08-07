from finite_set.base import FiniteSet, finite_set_from_unique_list_cardinality_is_length,
    finite_set_insert_cardinality_is_suc_of_not_contains,
    finite_set_insert_remove_of_not_contains,
    finite_set_remove_cardinality_is_pred_of_contains, fs_from_list, fs_insert
from data.basic.set import insert_contains_eq, list_set, list_set_contains_eq, subset_contains
from list import List, add_length, filter_contained_by_and, filter_contains_and,
    map
from list import map_filter_length_of_pointwise
from finite_set.list_subsets import insert_into_subset, list_subsets, list_subsets_exact,
    list_subsets_unique
from binary_words import filter_add, filter_preserves_unique, unique_cons_not_contains
from nat import Nat

numerals Nat

/// The list of members of `list_subsets(items)` whose finite cardinality is `k`.
define list_k_subsets[A](items: List[A], k: Nat) -> List[FiniteSet[A]] {
    list_subsets[A](items).filter(function(s: FiniteSet[A]) { s.cardinality_is(k) })
}

/// A finite set occurs in the filtered k-subset family exactly when it is a subset with cardinality `k`.
theorem list_k_subsets_exact[A](items: List[A], k: Nat, t: FiniteSet[A]) {
    list_k_subsets[A](items, k).contains(t) =
        (t.subset_eq(FiniteSet.from_list[A](items)) and t.cardinality_is(k))
} by {
    if list_k_subsets[A](items, k).contains(t) {
        filter_contained_by_and[FiniteSet[A]](
            list_subsets[A](items),
            function(s: FiniteSet[A]) { s.cardinality_is(k) },
            t
        )
        list_subsets[A](items).contains(t)
        list_subsets_exact[A](items, t)
        t.subset_eq(FiniteSet.from_list[A](items)) and t.cardinality_is(k)
    }
    if t.subset_eq(FiniteSet.from_list[A](items)) and t.cardinality_is(k) {
        list_subsets_exact[A](items, t)
        filter_contains_and[FiniteSet[A]](
            list_subsets[A](items),
            function(s: FiniteSet[A]) { s.cardinality_is(k) },
            t
        )
        list_k_subsets[A](items, k).contains(t)
    }
}

/// The finite set represented by the filtered k-subset list.
define finite_k_subsets_from_list[A](items: List[A], k: Nat) -> FiniteSet[FiniteSet[A]] {
    fs_from_list[FiniteSet[A]](list_k_subsets[A](items, k))
}

lemma finite_k_subsets_from_list_contains_eq[A](items: List[A], k: Nat, t: FiniteSet[A]) {
    finite_k_subsets_from_list[A](items, k).contains(t) = list_k_subsets[A](items, k).contains(t)
} by {
    fs_from_list[FiniteSet[A]](list_k_subsets[A](items, k)).underlying_set.contains(t) =
        list_set[FiniteSet[A]](list_k_subsets[A](items, k)).contains(t)
}

/// The finite filtered k-subset family contains exactly the k-cardinality subsets.
theorem finite_k_subsets_from_list_exact[A](items: List[A], k: Nat, t: FiniteSet[A]) {
    finite_k_subsets_from_list[A](items, k).contains(t) =
        (t.subset_eq(FiniteSet.from_list[A](items)) and t.cardinality_is(k))
} by {
    finite_k_subsets_from_list_contains_eq[A](items, k, t)
    list_k_subsets_exact[A](items, k, t)
}

/// Filtering `list_subsets` by cardinality preserves duplicate-freedom for duplicate-free bases.
theorem list_k_subsets_unique[A](items: List[A], k: Nat) {
    items.is_unique implies list_k_subsets[A](items, k).is_unique
} by {
    if items.is_unique {
        list_subsets_unique[A](items)
        filter_preserves_unique[FiniteSet[A]](
            list_subsets[A](items),
            function(s: FiniteSet[A]) { s.cardinality_is(k) }
        )
        list_k_subsets[A](items, k).is_unique
    }
}

/// The list-backed finite k-subset family is counted by its duplicate-free filtered list.
theorem finite_k_subsets_from_list_cardinality_is_length[A](items: List[A], k: Nat) {
    items.is_unique implies finite_k_subsets_from_list[A](items, k).cardinality_is(list_k_subsets[A](items, k).length)
} by {
    if items.is_unique {
        list_k_subsets_unique[A](items, k)
        finite_set_from_unique_list_cardinality_is_length[FiniteSet[A]](list_k_subsets[A](items, k))
        finite_k_subsets_from_list[A](items, k).cardinality_is(list_k_subsets[A](items, k).length)
    }
}

lemma finite_set_from_list_contains_eq_for_k[A](items: List[A], x: A) {
    FiniteSet.from_list[A](items).contains(x) = items.contains(x)
} by {
    FiniteSet.from_list[A](items).underlying_set = list_set[A](items)
    list_set_contains_eq(items, x)
}

lemma subset_of_from_list_not_contains_head_for_k[A](head: A, tail: List[A], t: FiniteSet[A]) {
    not tail.contains(head) and t.subset_eq(FiniteSet.from_list[A](tail)) implies not t.contains(head)
} by {
    if t.contains(head) {
        t.underlying_set.subset(FiniteSet.from_list[A](tail).underlying_set)
        subset_contains(t.underlying_set, FiniteSet.from_list[A](tail).underlying_set, head)
        FiniteSet.from_list[A](tail).contains(head)
        finite_set_from_list_contains_eq_for_k[A](tail, head)
        false
    }
}

lemma list_subsets_tail_members_avoid_head_for_k[A](head: A, tail: List[A], t: FiniteSet[A]) {
    not tail.contains(head) and list_subsets[A](tail).contains(t) implies not t.contains(head)
} by {
    list_subsets_exact[A](tail, t)
    subset_of_from_list_not_contains_head_for_k[A](head, tail, t)
}

lemma finite_set_insert_contains_for_binom[T](s: FiniteSet[T], item: T) {
    s.insert(item).contains(item)
} by {
    fs_insert(s, item).underlying_set = s.underlying_set.insert(item)
    insert_contains_eq(s.underlying_set, item, item)
}

lemma finite_set_insert_cardinality_suc_of_not_contains_for_binom[T](s: FiniteSet[T], item: T, k: Nat) {
    s.cardinality_is(k) and not s.contains(item) implies s.insert(item).cardinality_is(k.suc)
} by {
    if s.cardinality_is(k) and not s.contains(item) {
        finite_set_insert_cardinality_is_suc_of_not_contains[T](s, item, k)
        s.insert(item).cardinality_is(k.suc)
    }
}

lemma finite_set_insert_cardinality_pred_of_not_contains_for_binom[T](s: FiniteSet[T], item: T, k: Nat) {
    not s.contains(item) and s.insert(item).cardinality_is(k.suc) implies s.cardinality_is(k)
} by {
    if not s.contains(item) and s.insert(item).cardinality_is(k.suc) {
        finite_set_insert_contains_for_binom[T](s, item)
        s.insert(item).cardinality_is(k + Nat.1)
        finite_set_remove_cardinality_is_pred_of_contains[T](s.insert(item), item, k)
        s.insert(item).remove(item).cardinality_is(k)
        finite_set_insert_remove_of_not_contains[T](s, item)
        s.cardinality_is(k)
    }
}

lemma finite_set_insert_cardinality_suc_iff_of_not_contains_for_binom[T](s: FiniteSet[T], item: T, k: Nat) {
    not s.contains(item) implies (s.insert(item).cardinality_is(k.suc) = s.cardinality_is(k))
} by {
    if not s.contains(item) {
        if s.insert(item).cardinality_is(k.suc) {
            finite_set_insert_cardinality_pred_of_not_contains_for_binom[T](s, item, k)
            s.cardinality_is(k)
        }
        if s.cardinality_is(k) {
            finite_set_insert_cardinality_suc_of_not_contains_for_binom[T](s, item, k)
            s.insert(item).cardinality_is(k.suc)
        }
    }
}

lemma inserted_half_suc_length[A](head: A, tail: List[A], k: Nat) {
    not tail.contains(head) implies
    map[FiniteSet[A], FiniteSet[A]](list_subsets[A](tail), insert_into_subset[A](head)).filter(function(s: FiniteSet[A]) { s.cardinality_is(k.suc) }).length =
        list_subsets[A](tail).filter(function(s: FiniteSet[A]) { s.cardinality_is(k) }).length
} by {
    if not tail.contains(head) {
        forall(s: FiniteSet[A]) {
            if list_subsets[A](tail).contains(s) {
                list_subsets_tail_members_avoid_head_for_k[A](head, tail, s)
                finite_set_insert_cardinality_suc_iff_of_not_contains_for_binom[A](s, head, k)
                s.insert(head).cardinality_is(k.suc) = s.cardinality_is(k)
                insert_into_subset[A](head, s).cardinality_is(k.suc) = s.cardinality_is(k)
            }
        }
        map_filter_length_of_pointwise[FiniteSet[A], FiniteSet[A]](
            list_subsets[A](tail),
            insert_into_subset[A](head),
            function(u: FiniteSet[A]) { u.cardinality_is(k.suc) },
            function(u: FiniteSet[A]) { u.cardinality_is(k) }
        )
        map[FiniteSet[A], FiniteSet[A]](list_subsets[A](tail), insert_into_subset[A](head)).filter(function(u: FiniteSet[A]) { u.cardinality_is(k.suc) }).length =
            list_subsets[A](tail).filter(function(u: FiniteSet[A]) { u.cardinality_is(k) }).length
    }
}

/// Pascal step for nonzero k-subset list lengths over a duplicate-free cons base.
theorem list_k_subsets_cons_suc_length[A](head: A, tail: List[A], k: Nat) {
    List.cons[A](head, tail).is_unique implies
    list_k_subsets[A](List.cons[A](head, tail), k.suc).length =
        list_k_subsets[A](tail, k.suc).length + list_k_subsets[A](tail, k).length
} by {
    if List.cons[A](head, tail).is_unique {
        unique_cons_not_contains[A](head, tail)
        not tail.contains(head)
        let left = list_subsets[A](tail)
        let right = map[FiniteSet[A], FiniteSet[A]](left, insert_into_subset[A](head))
        list_subsets[A](List.cons[A](head, tail)) = left + right
        list_k_subsets[A](List.cons[A](head, tail), k.suc) =
            (left + right).filter(function(s: FiniteSet[A]) { s.cardinality_is(k.suc) })
        filter_add[FiniteSet[A]](left, right, function(s: FiniteSet[A]) { s.cardinality_is(k.suc) })
        (left + right).filter(function(s: FiniteSet[A]) { s.cardinality_is(k.suc) }) =
            left.filter(function(s: FiniteSet[A]) { s.cardinality_is(k.suc) }) +
            right.filter(function(s: FiniteSet[A]) { s.cardinality_is(k.suc) })
        list_k_subsets[A](tail, k.suc) = left.filter(function(s: FiniteSet[A]) { s.cardinality_is(k.suc) })
        list_k_subsets[A](tail, k) = left.filter(function(s: FiniteSet[A]) { s.cardinality_is(k) })
        inserted_half_suc_length[A](head, tail, k)
        add_length[FiniteSet[A]](
            left.filter(function(s: FiniteSet[A]) { s.cardinality_is(k.suc) }),
            right.filter(function(s: FiniteSet[A]) { s.cardinality_is(k.suc) })
        )
        list_k_subsets[A](List.cons[A](head, tail), k.suc).length =
            list_k_subsets[A](tail, k.suc).length + list_k_subsets[A](tail, k).length
    }
}
