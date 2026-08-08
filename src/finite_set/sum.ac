from finite_set.base import FiniteSet, fs_from_list, fs_empty, fs_insert, fs_union, finite_set_ext,
    finite_set_has_unique_list
from list import List
from list import map, unique_list_sum, add_contains_left, add_contains_right,
    not_contains_add
from list import sum, map_sum_add, sum_add, map_add, sum_scalar_mul, scalar_mul,
    map_map
from list import unique_same_contains_map_sum_eq, sum_map_remove_one,
    remove_one_unique, remove_one_unique_not_contains_self, remove_one_contains_other
from algebra.add_comm_monoid import AddCommMonoid
from algebra.add_semigroup import add_fn
from algebra.semigroup import mul_fn
from semiring import Semiring
from data.basic.functions import compose, function_extensionality
from data.basic.set import Set, list_set, list_set_contains_eq, insert_contains_eq,
    union_contains_eq
from nat import Nat

numerals Nat

/// The sum of `f` over the elements of a finite set.
/// Defined as the sum of `f` mapped over any unique-list representation of the set;
/// well-defined because two unique lists with the same membership give the same mapped sum.
let finite_set_sum[T, A: AddCommMonoid](s: FiniteSet[T], f: T -> A) -> result: A satisfy {
    exists(items: List[T]) {
        fs_from_list(items) = s and items.is_unique and sum[A](map(items, f)) = result
    }
} by {
    finite_set_has_unique_list(s)
    let items: List[T] satisfy {
        fs_from_list(items) = s and items.is_unique
    }
    let candidate = sum[A](map(items, f))
    fs_from_list(items) = s and items.is_unique and sum[A](map(items, f)) = candidate
    exists(items2: List[T]) {
        fs_from_list(items2) = s and items2.is_unique and sum[A](map(items2, f)) = candidate
    }
    exists(result: A) {
        exists(items2: List[T]) {
            fs_from_list(items2) = s and items2.is_unique and sum[A](map(items2, f)) = result
        }
    }
}

/// `finite_set_sum` is computed as the sum of `f` over any unique-list representation.
theorem finite_set_sum_eq_list_sum[T, A: AddCommMonoid](items: List[T], f: T -> A) {
    items.is_unique implies finite_set_sum(fs_from_list(items), f) = sum[A](map(items, f))
} by {
    if items.is_unique {
        let s = fs_from_list(items)
        // The defining property of finite_set_sum gives us a witness list.
        exists(witness: List[T]) {
            fs_from_list(witness) = s and witness.is_unique and sum[A](map(witness, f)) = finite_set_sum(s, f)
        }
        let items2: List[T] satisfy {
            fs_from_list(items2) = s and items2.is_unique and sum[A](map(items2, f)) = finite_set_sum(s, f)
        }
        let result = finite_set_sum(s, f)
        sum[A](map(items2, f)) = result
        fs_from_list(items).underlying_set = list_set(items)
        fs_from_list(items2).underlying_set = list_set(items2)
        fs_from_list(items) = fs_from_list(items2)
        list_set(items) = list_set(items2)
        forall(x: T) {
            list_set_contains_eq(items, x)
            list_set_contains_eq(items2, x)
            items.contains(x) = list_set(items).contains(x)
            list_set(items2).contains(x) = items2.contains(x)
            items.contains(x) = items2.contains(x)
        }
        unique_same_contains_map_sum_eq[T, A](items, items2, f)
        sum[A](map(items, f)) = sum[A](map(items2, f))
        sum[A](map(items2, f)) = result
        sum[A](map(items, f)) = result
        finite_set_sum(fs_from_list(items), f) = sum[A](map(items, f))
    }
}

/// The sum over the empty finite set is zero.
theorem finite_set_sum_empty[T, A: AddCommMonoid](f: T -> A) {
    finite_set_sum(fs_empty[T](Set[T].empty_set), f) = A.0
} by {
    let nil_list = List.nil[T]
    nil_list.is_unique
    map[T, A](nil_list, f) = List.nil[A]
    sum[A](List.nil[A]) = A.0
    sum[A](map(nil_list, f)) = A.0
    finite_set_sum_eq_list_sum[T, A](nil_list, f)
    finite_set_sum(fs_from_list(nil_list), f) = A.0
    fs_from_list(nil_list).underlying_set = list_set(nil_list)
    forall(x: T) {
        list_set_contains_eq(nil_list, x)
        list_set(nil_list).contains(x) = nil_list.contains(x)
        not nil_list.contains(x)
        not list_set(nil_list).contains(x)
        not Set[T].empty_set.contains(x)
        list_set(nil_list).contains(x) = Set[T].empty_set.contains(x)
    }
    list_set(nil_list) = Set[T].empty_set
    fs_from_list(nil_list).underlying_set = Set[T].empty_set
    fs_empty[T](Set[T].empty_set).underlying_set = Set[T].empty_set
    fs_from_list(nil_list).underlying_set = fs_empty[T](Set[T].empty_set).underlying_set
    finite_set_ext(fs_from_list(nil_list), fs_empty[T](Set[T].empty_set))
    fs_from_list(nil_list) = fs_empty[T](Set[T].empty_set)
    finite_set_sum(fs_empty[T](Set[T].empty_set), f) = A.0
}

/// Inserting an element not already in the set adds its value to the sum.
theorem finite_set_sum_insert[T, A: AddCommMonoid](s: FiniteSet[T], item: T, f: T -> A) {
    not s.contains(item) implies finite_set_sum(fs_insert(s, item), f) = f(item) + finite_set_sum(s, f)
} by {
    if not s.contains(item) {
        finite_set_has_unique_list(s)
        let items: List[T] satisfy {
            fs_from_list(items) = s and items.is_unique
        }
        fs_from_list(items).underlying_set = list_set(items)
        s.underlying_set = list_set(items)
        list_set_contains_eq(items, item)
        list_set(items).contains(item) = items.contains(item)
        s.underlying_set.contains(item) = items.contains(item)
        not items.contains(item)

        let cons_list = List.cons(item, items)
        items.unique = items
        cons_list.unique = List.cons(item, items.unique)
        cons_list.unique = List.cons(item, items)
        cons_list.unique = cons_list
        cons_list.is_unique

        finite_set_sum_eq_list_sum[T, A](cons_list, f)
        finite_set_sum(fs_from_list(cons_list), f) = sum[A](map(cons_list, f))
        map(cons_list, f) = List.cons(f(item), map(items, f))
        sum[A](List.cons(f(item), map(items, f))) = f(item) + sum[A](map(items, f))
        sum[A](map(cons_list, f)) = f(item) + sum[A](map(items, f))

        forall(x: T) {
            list_set_contains_eq(cons_list, x)
            list_set_contains_eq(items, x)
            insert_contains_eq(s.underlying_set, item, x)
            list_set(cons_list).contains(x) = cons_list.contains(x)
            if x = item {
                cons_list.contains(x)
                cons_list.contains(x) = (x = item or items.contains(x))
            } else {
                item != x
                if items.contains(x) {
                    List.cons(item, items).contains(x)
                    cons_list.contains(x)
                }
                if cons_list.contains(x) {
                    items.contains(x)
                }
                cons_list.contains(x) = items.contains(x)
                cons_list.contains(x) = (x = item or items.contains(x))
            }
            s.underlying_set.insert(item).contains(x) = (x = item or s.underlying_set.contains(x))
            s.underlying_set.contains(x) = list_set(items).contains(x)
            list_set(items).contains(x) = items.contains(x)
            list_set(cons_list).contains(x) = s.underlying_set.insert(item).contains(x)
        }
        list_set(cons_list) = s.underlying_set.insert(item)
        fs_from_list(cons_list).underlying_set = list_set(cons_list)
        fs_insert(s, item).underlying_set = s.underlying_set.insert(item)
        fs_from_list(cons_list).underlying_set = fs_insert(s, item).underlying_set
        finite_set_ext(fs_from_list(cons_list), fs_insert(s, item))
        fs_from_list(cons_list) = fs_insert(s, item)

        finite_set_sum(fs_insert(s, item), f) = sum[A](map(cons_list, f))
        finite_set_sum_eq_list_sum[T, A](items, f)
        finite_set_sum(s, f) = sum[A](map(items, f))
        finite_set_sum(fs_insert(s, item), f) = f(item) + finite_set_sum(s, f)
    }
}

/// Sum is additive in the function argument.
theorem finite_set_sum_add[T, A: AddCommMonoid](s: FiniteSet[T], f: T -> A, g: T -> A) {
    finite_set_sum(s, f) + finite_set_sum(s, g) = finite_set_sum(s, add_fn(f, g))
} by {
    finite_set_has_unique_list(s)
    let items: List[T] satisfy {
        fs_from_list(items) = s and items.is_unique
    }
    finite_set_sum_eq_list_sum[T, A](items, f)
    finite_set_sum_eq_list_sum[T, A](items, g)
    finite_set_sum_eq_list_sum[T, A](items, add_fn(f, g))
    finite_set_sum(fs_from_list(items), f) = sum[A](map(items, f))
    finite_set_sum(fs_from_list(items), g) = sum[A](map(items, g))
    finite_set_sum(fs_from_list(items), add_fn(f, g)) = sum[A](map(items, add_fn(f, g)))
    finite_set_sum(s, f) = sum[A](map(items, f))
    finite_set_sum(s, g) = sum[A](map(items, g))
    finite_set_sum(s, add_fn(f, g)) = sum[A](map(items, add_fn(f, g)))
    map_sum_add[T, A](items, f, g)
    sum[A](map(items, f)) + sum[A](map(items, g)) = sum[A](map(items, add_fn(f, g)))
    finite_set_sum(s, f) + finite_set_sum(s, g) = finite_set_sum(s, add_fn(f, g))
}

/// The sum over a disjoint union splits as the sum over each part.
theorem finite_set_sum_disjoint_union[T, A: AddCommMonoid](a: FiniteSet[T], b: FiniteSet[T], f: T -> A) {
    a.is_disjoint(b) implies
        finite_set_sum(fs_union(a, b), f) = finite_set_sum(a, f) + finite_set_sum(b, f)
} by {
    if a.is_disjoint(b) {
        finite_set_has_unique_list(a)
        let la: List[T] satisfy {
            fs_from_list(la) = a and la.is_unique
        }
        finite_set_has_unique_list(b)
        let lb: List[T] satisfy {
            fs_from_list(lb) = b and lb.is_unique
        }
        fs_from_list(la).underlying_set = list_set(la)
        a.underlying_set = list_set(la)
        fs_from_list(lb).underlying_set = list_set(lb)
        b.underlying_set = list_set(lb)

        forall(x: T) {
            list_set_contains_eq(la, x)
            list_set_contains_eq(lb, x)
            la.contains(x) = list_set(la).contains(x)
            lb.contains(x) = list_set(lb).contains(x)
            la.contains(x) = a.underlying_set.contains(x)
            lb.contains(x) = b.underlying_set.contains(x)
            not (a.underlying_set.contains(x) and b.underlying_set.contains(x))
            not (la.contains(x) and lb.contains(x))
        }
        unique_list_sum[T](la, lb)
        let combined = la + lb
        combined.is_unique

        forall(x: T) {
            list_set_contains_eq(combined, x)
            list_set(combined).contains(x) = combined.contains(x)
            if combined.contains(x) {
                not_contains_add[T](la, lb, x)
                la.contains(x) or lb.contains(x)
            }
            if la.contains(x) {
                add_contains_left[T](la, lb, x)
                combined.contains(x)
            }
            if lb.contains(x) {
                add_contains_right[T](la, lb, x)
                combined.contains(x)
            }
            combined.contains(x) = (la.contains(x) or lb.contains(x))
            la.contains(x) = a.underlying_set.contains(x)
            lb.contains(x) = b.underlying_set.contains(x)
            union_contains_eq(a.underlying_set, b.underlying_set, x)
            a.underlying_set.union(b.underlying_set).contains(x) = (a.underlying_set.contains(x) or b.underlying_set.contains(x))
            list_set(combined).contains(x) = a.underlying_set.union(b.underlying_set).contains(x)
        }
        list_set(combined) = a.underlying_set.union(b.underlying_set)
        fs_from_list(combined).underlying_set = list_set(combined)
        fs_union(a, b).underlying_set = a.underlying_set.union(b.underlying_set)
        fs_from_list(combined).underlying_set = fs_union(a, b).underlying_set
        finite_set_ext(fs_from_list(combined), fs_union(a, b))
        fs_from_list(combined) = fs_union(a, b)

        finite_set_sum_eq_list_sum[T, A](combined, f)
        finite_set_sum(fs_from_list(combined), f) = sum[A](map(combined, f))
        finite_set_sum(fs_union(a, b), f) = sum[A](map(combined, f))

        map_add[T, A](la, lb, f)
        map(combined, f) = map(la, f) + map(lb, f)
        sum_add[A](map(la, f), map(lb, f))
        sum[A](map(la, f) + map(lb, f)) = sum[A](map(la, f)) + sum[A](map(lb, f))
        sum[A](map(combined, f)) = sum[A](map(la, f)) + sum[A](map(lb, f))

        finite_set_sum_eq_list_sum[T, A](la, f)
        finite_set_sum_eq_list_sum[T, A](lb, f)
        finite_set_sum(a, f) = sum[A](map(la, f))
        finite_set_sum(b, f) = sum[A](map(lb, f))

        finite_set_sum(fs_union(a, b), f) = finite_set_sum(a, f) + finite_set_sum(b, f)
    }
}

/// Scalar multiplication distributes over a finite sum:
/// `c * finite_set_sum(s, f) = finite_set_sum(s, x -> c * f(x))`.
theorem finite_set_sum_scalar_mul[T, S: Semiring](c: S, s: FiniteSet[T], f: T -> S) {
    c * finite_set_sum(s, f) = finite_set_sum(s, mul_fn(c, f))
} by {
    finite_set_has_unique_list(s)
    let items: List[T] satisfy {
        fs_from_list(items) = s and items.is_unique
    }
    finite_set_sum_eq_list_sum[T, S](items, f)
    finite_set_sum(s, f) = sum[S](map(items, f))
    finite_set_sum_eq_list_sum[T, S](items, mul_fn(c, f))
    finite_set_sum(s, mul_fn(c, f)) = sum[S](map(items, mul_fn(c, f)))

    sum_scalar_mul[S](c, map(items, f))
    c * sum[S](map(items, f)) = sum[S](map(map(items, f), scalar_mul(c)))

    map_map[T, S, S](items, f, scalar_mul(c))
    map(map(items, f), scalar_mul(c)) = map(items, compose(scalar_mul(c), f))

    forall(x: T) {
        compose[T, S, S](scalar_mul(c), f, x) = scalar_mul(c, f(x))
        scalar_mul(c, f(x)) = c * f(x)
        mul_fn(c, f, x) = c * f(x)
        compose[T, S, S](scalar_mul(c), f, x) = mul_fn(c, f, x)
    }
    function_extensionality[T, S](compose(scalar_mul(c), f), mul_fn(c, f))
    compose(scalar_mul(c), f) = mul_fn(c, f)

    map(items, compose(scalar_mul(c), f)) = map(items, mul_fn(c, f))
    map(map(items, f), scalar_mul(c)) = map(items, mul_fn(c, f))
    sum[S](map(map(items, f), scalar_mul(c))) = sum[S](map(items, mul_fn(c, f)))
    c * sum[S](map(items, f)) = sum[S](map(items, mul_fn(c, f)))
    c * finite_set_sum(s, f) = finite_set_sum(s, mul_fn(c, f))
}
