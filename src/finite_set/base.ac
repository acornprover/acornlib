from nat import Nat, add_assoc, add_cancels_left, add_comm, add_imp_sub, add_sub,
    lte_suc_suc, sub_lt
from list import List, list_contains_has_greatest, list_contains_has_least,
    list_contains_has_maximal, list_contains_has_minimal
from list import pigeonhole_map_into_list
from data.basic.functions import compose, identity_fn, Inhabited, inverse_fn, is_bijection_fn,
    is_injective_fn, bijection_fn_is_injective
from data.basic.logic import exists_intro, and_assoc, and_or_distrib_right, eq_false_intro
from order import LinearOrder, PartialOrder, lte_refl, lte_antisymm, lte_trans, min_is_one, max_is_one,
    min_lte_left, min_lte_right, lte_max_left, lte_max_right
from data.basic.set import Set, set_ext, finite_constraint, functional_insert, functional_remove,
    empty_set_contains_eq, singleton_contains_eq, singleton_set_cardinality_is_one,
    subset_contains, subset_refl, subset_trans,
    subset_antisymm, empty_set_is_always_subset, union_contains_eq,
    intersection_contains_eq, difference_contains_eq, sets_subset_union,
    sets_subset_contain_union, sets_subset_intersection, set_supset_contains_intersection,
    union_comm, union_idemp, union_with_empty_is_self, intersection_comm,
    intersection_idemp, intersection_with_empty_is_empty, insert_eq_union_singleton,
    list_set, list_set_is_finite, list_set_cardinality_at_most_length,
    unique_list_set_cardinality_is_length, list_set_filter_extracts_subset,
    list_set_contains_eq, set_has_exact_containing_list, set_cardinality_has_exact_unique_list,
    set_image, maps_into_set_image, set_image_contains_witness,
    set_image_is_finite_of_finite, set_image_contains_inverse_of_bijection,
    set_inverse_contains_imp_image_contains, set_image_empty, set_image_singleton,
    set_image_monotone,
    set_image_cardinality_at_most, set_image_cardinality_is_of_injective,
    insert_cardinality_is_suc_of_not_contains, remove_insert_cardinality_is_suc_of_contains,
    remove_cardinality_is_of_not_contains, remove_cardinality_is_pred_of_contains,
    insert_then_remove, intersection_cardinality_is_of_subset_left,
    intersection_cardinality_is_of_subset_right,
    difference_cardinality_is_of_disjoint, difference_cardinality_is_zero_of_subset,
    disjoint_union_is_length, union_cardinality_with_difference,
    intersection_is_disjoint_difference, intersection_union_difference_is_self,
    intersection_difference_cardinality_adds, difference_cardinality_is_sub_intersection,
    intersection_cardinality_is_sub_difference, inclusion_exclusion,
    cardinality_always_exists, cardinality_is_well_defined, cardinality_is_smallest_cardinality,
    cardinality_is_implies_is_finite
from data.basic.witness import exists_unique, exists_unique_exists, exists_unique_eq,
    set_witness_predicate, set_witness_contains, set_witness_property,
    set_witness_intro, choose_from_set_or_default, choose_from_set_or_default_spec,
    choose_from_set_or_default_contains, choose_from_set_or_default_property,
    choose_from_set_or_default_eq_default

numerals Nat

/// A finite set is a `Set` bundled with a finiteness proof.
structure FiniteSet[T] {
    /// The underlying set.
    underlying_set: Set[T]
} constraint { 
    underlying_set.is_finite
}

let fs_empty[T](s: Set[T]) -> result: FiniteSet[T] satisfy {
    FiniteSet.new(Set[T].empty_set) = Option.some(result)
}

let fs_from_list[T](items: List[T]) -> result: FiniteSet[T] satisfy {
    FiniteSet.new(list_set(items)) = Option.some(result)
} by {
    list_set_is_finite(items)
}

let fs_insert[T](s: FiniteSet[T], item: T) -> result: FiniteSet[T] satisfy {
    FiniteSet.new(s.underlying_set.insert(item)) = Option.some(result)
} by {
    let f = s.underlying_set.contains
    finite_constraint(f)
    finite_constraint(functional_insert(f, item))
    let inserted = s.underlying_set.insert(item)
    forall(x: T) {
        inserted.contains(x) = functional_insert(f, item, x)
    }
    inserted.contains = functional_insert(f, item)
    inserted.is_finite
}

let fs_remove[T](s: FiniteSet[T], item: T) -> result: FiniteSet[T] satisfy {
    FiniteSet.new(s.underlying_set.remove(item)) = Option.some(result)
} by {
    let f = s.underlying_set.contains
    finite_constraint(f)
    finite_constraint(functional_remove(f, item))
    let removed = s.underlying_set.remove(item)
    forall(x: T) {
        removed.contains(x) = functional_remove(f, item, x)
    }
    removed.contains = functional_remove(f, item)
    removed.is_finite
}

let fs_union[T](a: FiniteSet[T], b: FiniteSet[T]) -> result: FiniteSet[T] satisfy {
    FiniteSet.new(a.underlying_set.union(b.underlying_set)) = Option.some(result)
}

let fs_intersection[T](a: FiniteSet[T], b: FiniteSet[T]) -> result: FiniteSet[T] satisfy {
    FiniteSet.new(a.underlying_set.intersection(b.underlying_set)) = Option.some(result)
}

let fs_difference[T](a: FiniteSet[T], b: FiniteSet[T]) -> result: FiniteSet[T] satisfy {
    FiniteSet.new(a.underlying_set.difference(b.underlying_set)) = Option.some(result)
}

let fs_image[T, U](s: FiniteSet[T], f: T -> U) -> result: FiniteSet[U] satisfy {
    FiniteSet.new(set_image(s.underlying_set, f)) = Option.some(result)
} by {
    set_image_is_finite_of_finite(s.underlying_set, f)
}

/// Finite set extensionality reduces equality to equality of the underlying sets.
theorem finite_set_ext[T](a: FiniteSet[T], b: FiniteSet[T]) {
    a.underlying_set = b.underlying_set implies a = b
}

/// Finite sets with the same elements are equal.
theorem finite_set_ext_contains[T](a: FiniteSet[T], b: FiniteSet[T]) {
    (forall(x: T) {
        a.underlying_set.contains(x) = b.underlying_set.contains(x)
    }) implies a = b
} by {
    if forall(x: T) {
        a.underlying_set.contains(x) = b.underlying_set.contains(x)
    } {
        set_ext(a.underlying_set, b.underlying_set)
        finite_set_ext(a, b)
    }
}

/// Equal finite sets have equal underlying sets.
theorem finite_set_eq_underlying_set[T](a: FiniteSet[T], b: FiniteSet[T]) {
    a = b implies a.underlying_set = b.underlying_set
}

/// Equal finite sets have equal underlying membership at every element.
theorem finite_set_eq_underlying_contains_at[T](a: FiniteSet[T], b: FiniteSet[T], x: T) {
    a = b implies a.underlying_set.contains(x) = b.underlying_set.contains(x)
} by {
    if a = b {
        a.underlying_set.contains(x) = b.underlying_set.contains(x)
    }
}

/// Equality of finite sets transports predicates on finite sets.
theorem finite_set_eq_transport_predicate[T](p: FiniteSet[T] -> Bool, a: FiniteSet[T], b: FiniteSet[T]) {
    a = b and p(a) implies p(b)
}

/// Equality of finite sets transports predicates on finite sets in the reverse direction.
theorem finite_set_eq_transport_predicate_rev[T](p: FiniteSet[T] -> Bool, a: FiniteSet[T], b: FiniteSet[T]) {
    a = b and p(b) implies p(a)
}

/// Equality of underlying sets determines equality of finite sets.
theorem finite_set_eq_of_underlying_set_eq[T](a: FiniteSet[T], b: FiniteSet[T]) {
    a.underlying_set = b.underlying_set implies a = b
} by {
    if a.underlying_set = b.underlying_set {
        finite_set_ext(a, b)
    }
}

attributes FiniteSet[T] {
    /// Finite set extensionality from equality of the underlying sets.
    let ext = finite_set_ext[T]

    /// Access the underlying set.
    define as_set(self) -> Set[T] {
        self.underlying_set
    }

    /// Membership predicate.
    define contains(self, x: T) -> Bool {
        self.underlying_set.contains(x)
    }

    /// The empty finite set.
    let empty: FiniteSet[T] = fs_empty(Set[T].empty_set)

    /// The finite set whose elements are the elements of the list.
    let from_list: List[T] -> FiniteSet[T] = fs_from_list

    /// Insert preserves finiteness.
    let insert: (FiniteSet[T], T) -> FiniteSet[T] = fs_insert

    /// Remove preserves finiteness.
    let remove: (FiniteSet[T], T) -> FiniteSet[T] = fs_remove

    /// Subset relation lifted from sets.
    define subset_eq(self, other: FiniteSet[T]) -> Bool {
        self.underlying_set.subset(other.underlying_set)
    }

    /// Superset relation lifted from sets.
    define superset_eq(self, other: FiniteSet[T]) -> Bool {
        self.underlying_set.superset(other.underlying_set)
    }

    /// Union of finite sets.
    let union: (FiniteSet[T], FiniteSet[T]) -> FiniteSet[T] = fs_union

    /// Intersection of finite sets.
    let intersection: (FiniteSet[T], FiniteSet[T]) -> FiniteSet[T] = fs_intersection

    /// Difference of finite sets.
    let difference: (FiniteSet[T], FiniteSet[T]) -> FiniteSet[T] = fs_difference

    /// Image of a finite set under a function.
    define image[U](self, f: T -> U) -> FiniteSet[U] {
        fs_image(self, f)
    }

    /// Disjointness predicate.
    define is_disjoint(self, other: FiniteSet[T]) -> Bool {
        self.underlying_set.is_disjoint(other.underlying_set)
    }

    /// Empty predicate.
    define is_empty(self) -> Bool {
        self.underlying_set.is_empty
    }

    /// Cardinality helper lifted from sets.
    define cardinality_at_most(self, n: Nat) -> Bool {
        self.underlying_set.cardinality_at_most(n)
    }

    /// True if the cardinality equals n.
    define cardinality_is(self, n: Nat) -> Bool {
        self.underlying_set.cardinality_is(n)
    }
}

/// The empty finite set has no elements.
theorem finite_set_empty_contains_eq[T](x: T) {
    FiniteSet.empty[T].contains(x) = false
} by {
    FiniteSet.empty[T].underlying_set = Set[T].empty_set
    empty_set_contains_eq[T](x)
}

/// Membership in a singleton finite set is equality with its element.
theorem finite_set_singleton_contains_eq[T](a: T, x: T) {
    fs_insert(FiniteSet.empty[T], a).contains(x) = (x = a)
} by {
    fs_insert(FiniteSet.empty[T], a).underlying_set =
        FiniteSet.empty[T].underlying_set.insert(a)
    FiniteSet.empty[T].underlying_set = Set[T].empty_set
    insert_eq_union_singleton(Set[T].empty_set, a)
    union_comm(Set[T].empty_set, Set[T].singleton(a))
    union_with_empty_is_self(Set[T].singleton(a))
    fs_insert(FiniteSet.empty[T], a).underlying_set = Set[T].singleton(a)
    singleton_contains_eq(a, x)
    fs_insert(FiniteSet.empty[T], a).contains(x) = (x = a)
}

/// Membership in a finite-set union is membership in either finite set.
theorem finite_set_union_contains_eq[T](a: FiniteSet[T], b: FiniteSet[T], x: T) {
    fs_union(a, b).contains(x) = (a.contains(x) or b.contains(x))
} by {
    fs_union(a, b).underlying_set = a.underlying_set.union(b.underlying_set)
    union_contains_eq(a.underlying_set, b.underlying_set, x)
}

/// Membership in a finite-set intersection is membership in both finite sets.
theorem finite_set_intersection_contains_eq[T](a: FiniteSet[T], b: FiniteSet[T], x: T) {
    fs_intersection(a, b).contains(x) = (a.contains(x) and b.contains(x))
} by {
    fs_intersection(a, b).underlying_set = a.underlying_set.intersection(b.underlying_set)
    intersection_contains_eq(a.underlying_set, b.underlying_set, x)
}

/// Membership in a finite-set difference is membership in the left finite set and not the right one.
theorem finite_set_difference_contains_eq[T](a: FiniteSet[T], b: FiniteSet[T], x: T) {
    fs_difference(a, b).contains(x) = (a.contains(x) and not b.contains(x))
} by {
    fs_difference(a, b).underlying_set = a.underlying_set.difference(b.underlying_set)
    difference_contains_eq(a.underlying_set, b.underlying_set, x)
}

/// A finite subset carries membership into the finite superset.
theorem finite_set_subset_contains[T](a: FiniteSet[T], b: FiniteSet[T], x: T) {
    a.subset_eq(b) and a.contains(x) implies b.contains(x)
} by {
    if a.subset_eq(b) and a.contains(x) {
        a.underlying_set.contains(x)
        subset_contains(a.underlying_set, b.underlying_set, x)
        b.underlying_set.contains(x)
        b.contains(x)
    }
}

/// Every finite set is a subset of itself.
theorem finite_set_subset_refl[T](s: FiniteSet[T]) {
    s.subset_eq(s)
} by {
    subset_refl(s.underlying_set)
}

/// Finite-set subset is transitive.
theorem finite_set_subset_trans[T](a: FiniteSet[T], b: FiniteSet[T], c: FiniteSet[T]) {
    a.subset_eq(b) and b.subset_eq(c) implies a.subset_eq(c)
} by {
    if a.subset_eq(b) and b.subset_eq(c) {
        b.underlying_set.subset(c.underlying_set)
        subset_trans(a.underlying_set, b.underlying_set, c.underlying_set)
        a.subset_eq(c)
    }
}

/// Finite-set subset is antisymmetric.
theorem finite_set_subset_antisymm[T](a: FiniteSet[T], b: FiniteSet[T]) {
    a.subset_eq(b) and b.subset_eq(a) implies a = b
} by {
    if a.subset_eq(b) and b.subset_eq(a) {
        b.underlying_set.subset(a.underlying_set)
        subset_antisymm(a.underlying_set, b.underlying_set)
        finite_set_ext(a, b)
        a = b
    }
}

/// The empty finite set is contained in every finite set.
theorem finite_set_empty_subset[T](s: FiniteSet[T]) {
    FiniteSet.empty[T].subset_eq(s)
} by {
    FiniteSet.empty[T].underlying_set = Set[T].empty_set
    empty_set_is_always_subset(s.underlying_set)
}

/// The left operand is contained in a finite-set union.
theorem finite_set_subset_union_left[T](a: FiniteSet[T], b: FiniteSet[T]) {
    a.subset_eq(fs_union(a, b))
} by {
    fs_union(a, b).underlying_set = a.underlying_set.union(b.underlying_set)
    sets_subset_union(a.underlying_set, b.underlying_set)
    a.underlying_set.subset(a.underlying_set.union(b.underlying_set))
    a.subset_eq(fs_union(a, b))
}

/// The right operand is contained in a finite-set union.
theorem finite_set_subset_union_right[T](a: FiniteSet[T], b: FiniteSet[T]) {
    b.subset_eq(fs_union(a, b))
} by {
    fs_union(a, b).underlying_set = a.underlying_set.union(b.underlying_set)
    sets_subset_union(a.underlying_set, b.underlying_set)
    b.underlying_set.subset(a.underlying_set.union(b.underlying_set))
    b.subset_eq(fs_union(a, b))
}

/// A finite-set union is contained in a finite set exactly when both parts are contained in it.
theorem finite_set_union_subset_eq[T](a: FiniteSet[T], b: FiniteSet[T], c: FiniteSet[T]) {
    fs_union(a, b).subset_eq(c) = (a.subset_eq(c) and b.subset_eq(c))
} by {
    if fs_union(a, b).subset_eq(c) {
        fs_union(a, b).underlying_set = a.underlying_set.union(b.underlying_set)
        sets_subset_union(a.underlying_set, b.underlying_set)
        subset_trans(a.underlying_set, a.underlying_set.union(b.underlying_set), c.underlying_set)
        subset_trans(b.underlying_set, a.underlying_set.union(b.underlying_set), c.underlying_set)
        a.underlying_set.subset(c.underlying_set)
        b.underlying_set.subset(c.underlying_set)
        b.subset_eq(c)
        a.subset_eq(c) and b.subset_eq(c)
    }
    if a.subset_eq(c) and b.subset_eq(c) {
        b.underlying_set.subset(c.underlying_set)
        sets_subset_contain_union(a.underlying_set, b.underlying_set, c.underlying_set)
        a.underlying_set.union(b.underlying_set).subset(c.underlying_set)
        fs_union(a, b).underlying_set = a.underlying_set.union(b.underlying_set)
        fs_union(a, b).subset_eq(c)
    }
}

/// A finite set is contained in an intersection exactly when it is contained in both finite sets.
theorem finite_set_subset_intersection_eq[T](a: FiniteSet[T], b: FiniteSet[T], c: FiniteSet[T]) {
    c.subset_eq(fs_intersection(a, b)) = (c.subset_eq(a) and c.subset_eq(b))
} by {
    if c.subset_eq(fs_intersection(a, b)) {
        fs_intersection(a, b).underlying_set = a.underlying_set.intersection(b.underlying_set)
        sets_subset_intersection(a.underlying_set, b.underlying_set)
        subset_trans(c.underlying_set, a.underlying_set.intersection(b.underlying_set), a.underlying_set)
        subset_trans(c.underlying_set, a.underlying_set.intersection(b.underlying_set), b.underlying_set)
        c.underlying_set.subset(a.underlying_set)
        c.underlying_set.subset(b.underlying_set)
        c.subset_eq(b)
        c.subset_eq(a) and c.subset_eq(b)
    }
    if c.subset_eq(a) and c.subset_eq(b) {
        b.underlying_set.superset(c.underlying_set)
        set_supset_contains_intersection(a.underlying_set, b.underlying_set, c.underlying_set)
        a.underlying_set.intersection(b.underlying_set).superset(c.underlying_set)
        c.underlying_set.subset(a.underlying_set.intersection(b.underlying_set))
        fs_intersection(a, b).underlying_set = a.underlying_set.intersection(b.underlying_set)
        c.subset_eq(fs_intersection(a, b))
    }
}

/// Finite-set union is commutative.
theorem finite_set_union_comm[T](a: FiniteSet[T], b: FiniteSet[T]) {
    fs_union(a, b) = fs_union(b, a)
} by {
    fs_union(a, b).underlying_set = a.underlying_set.union(b.underlying_set)
    fs_union(b, a).underlying_set = b.underlying_set.union(a.underlying_set)
    union_comm(a.underlying_set, b.underlying_set)
    finite_set_ext(fs_union(a, b), fs_union(b, a))
}

/// Finite-set intersection is commutative.
theorem finite_set_intersection_comm[T](a: FiniteSet[T], b: FiniteSet[T]) {
    fs_intersection(a, b) = fs_intersection(b, a)
} by {
    fs_intersection(a, b).underlying_set = a.underlying_set.intersection(b.underlying_set)
    fs_intersection(b, a).underlying_set = b.underlying_set.intersection(a.underlying_set)
    intersection_comm(a.underlying_set, b.underlying_set)
    finite_set_ext(fs_intersection(a, b), fs_intersection(b, a))
}

/// Finite-set intersection is associative.
theorem finite_set_intersection_assoc[T](a: FiniteSet[T], b: FiniteSet[T], c: FiniteSet[T]) {
    fs_intersection(fs_intersection(a, b), c) =
        fs_intersection(a, fs_intersection(b, c))
} by {
    forall(x: T) {
        finite_set_intersection_contains_eq(fs_intersection(a, b), c, x)
        finite_set_intersection_contains_eq(a, b, x)
        finite_set_intersection_contains_eq(a, fs_intersection(b, c), x)
        finite_set_intersection_contains_eq(b, c, x)
        and_assoc(a.contains(x), b.contains(x), c.contains(x))
        fs_intersection(fs_intersection(a, b), c).contains(x) =
            fs_intersection(a, fs_intersection(b, c)).contains(x)
    }
    finite_set_ext_contains(
        fs_intersection(fs_intersection(a, b), c),
        fs_intersection(a, fs_intersection(b, c)))
}

/// Intersection distributes over finite-set union.
theorem finite_set_intersection_union_distrib[T](
    a: FiniteSet[T], b: FiniteSet[T], c: FiniteSet[T]
) {
    fs_intersection(fs_union(a, b), c) =
        fs_union(fs_intersection(a, c), fs_intersection(b, c))
} by {
    forall(x: T) {
        finite_set_intersection_contains_eq(fs_union(a, b), c, x)
        finite_set_union_contains_eq(a, b, x)
        finite_set_union_contains_eq(fs_intersection(a, c), fs_intersection(b, c), x)
        finite_set_intersection_contains_eq(a, c, x)
        finite_set_intersection_contains_eq(b, c, x)
        and_or_distrib_right(a.contains(x), b.contains(x), c.contains(x))
        fs_intersection(fs_union(a, b), c).contains(x) =
            fs_union(fs_intersection(a, c), fs_intersection(b, c)).contains(x)
    }
    finite_set_ext_contains(
        fs_intersection(fs_union(a, b), c),
        fs_union(fs_intersection(a, c), fs_intersection(b, c)))
}

/// Intersecting two intersections with a common finite set gives their triple intersection.
theorem finite_set_pair_intersections[T](
    a: FiniteSet[T], b: FiniteSet[T], c: FiniteSet[T]
) {
    fs_intersection(fs_intersection(a, c), fs_intersection(b, c)) =
        fs_intersection(fs_intersection(a, b), c)
} by {
    forall(x: T) {
        finite_set_intersection_contains_eq(fs_intersection(a, c), fs_intersection(b, c), x)
        finite_set_intersection_contains_eq(a, c, x)
        finite_set_intersection_contains_eq(b, c, x)
        finite_set_intersection_contains_eq(fs_intersection(a, b), c, x)
        finite_set_intersection_contains_eq(a, b, x)
        if a.contains(x) {
            if b.contains(x) {
                if c.contains(x) {
                    fs_intersection(fs_intersection(a, c), fs_intersection(b, c)).contains(x) =
                        fs_intersection(fs_intersection(a, b), c).contains(x)
                } else {
                    fs_intersection(fs_intersection(a, c), fs_intersection(b, c)).contains(x) =
                        fs_intersection(fs_intersection(a, b), c).contains(x)
                }
            } else {
                fs_intersection(fs_intersection(a, c), fs_intersection(b, c)).contains(x) =
                    fs_intersection(fs_intersection(a, b), c).contains(x)
            }
        } else {
            fs_intersection(fs_intersection(a, c), fs_intersection(b, c)).contains(x) =
                fs_intersection(fs_intersection(a, b), c).contains(x)
        }
        fs_intersection(fs_intersection(a, c), fs_intersection(b, c)).contains(x) =
            fs_intersection(fs_intersection(a, b), c).contains(x)
    }
    finite_set_ext_contains(
        fs_intersection(fs_intersection(a, c), fs_intersection(b, c)),
        fs_intersection(fs_intersection(a, b), c))
}

/// Finite-set union is idempotent.
theorem finite_set_union_idemp[T](s: FiniteSet[T]) {
    fs_union(s, s) = s
} by {
    fs_union(s, s).underlying_set = s.underlying_set.union(s.underlying_set)
    union_idemp(s.underlying_set)
    finite_set_ext(fs_union(s, s), s)
}

/// Finite-set intersection is idempotent.
theorem finite_set_intersection_idemp[T](s: FiniteSet[T]) {
    fs_intersection(s, s) = s
} by {
    fs_intersection(s, s).underlying_set = s.underlying_set.intersection(s.underlying_set)
    intersection_idemp(s.underlying_set)
    finite_set_ext(fs_intersection(s, s), s)
}

/// Union with the empty finite set preserves a finite set.
theorem finite_set_union_empty_right[T](s: FiniteSet[T]) {
    fs_union(s, FiniteSet.empty[T]) = s
} by {
    fs_union(s, FiniteSet.empty[T]).underlying_set =
        s.underlying_set.union(FiniteSet.empty[T].underlying_set)
    FiniteSet.empty[T].underlying_set = Set[T].empty_set
    union_with_empty_is_self(s.underlying_set)
    finite_set_ext(fs_union(s, FiniteSet.empty[T]), s)
}

/// Intersection with the empty finite set is the empty finite set.
theorem finite_set_intersection_empty_right[T](s: FiniteSet[T]) {
    fs_intersection(s, FiniteSet.empty[T]) = FiniteSet.empty[T]
} by {
    fs_intersection(s, FiniteSet.empty[T]).underlying_set =
        s.underlying_set.intersection(FiniteSet.empty[T].underlying_set)
    FiniteSet.empty[T].underlying_set = Set[T].empty_set
    intersection_with_empty_is_empty(s.underlying_set)
    finite_set_ext(fs_intersection(s, FiniteSet.empty[T]), FiniteSet.empty[T])
}

/// Membership in a finite-set image is witnessed by a preimage in the finite set.
theorem finite_set_image_contains_eq[T, U](s: FiniteSet[T], f: T -> U, y: U) {
    fs_image(s, f).contains(y) = exists(x: T) {
        s.contains(x) and y = f(x)
    }
} by {
    if fs_image(s, f).contains(y) {
        fs_image(s, f).underlying_set = set_image(s.underlying_set, f)
        set_image_contains_witness(s.underlying_set, f, y)
        let x: T satisfy {
            s.underlying_set.contains(x) and y = f(x)
        }
        fs_image(s, f).contains(y) = exists(z: T) {
            s.contains(z) and y = f(z)
        }
    } else {
        if exists(x: T) { s.contains(x) and y = f(x) } {
            let x: T satisfy {
                s.contains(x) and y = f(x)
            }
            s.underlying_set.contains(x)
            maps_into_set_image(s.underlying_set, f, x)
            set_image(s.underlying_set, f).contains(f(x))
            set_image(s.underlying_set, f).contains(y)
            fs_image(s, f).underlying_set = set_image(s.underlying_set, f)
            false
        }
        fs_image(s, f).contains(y) = exists(x: T) {
            s.contains(x) and y = f(x)
        }
    }
}

/// Membership in a finite-set image provides a preimage witness in the finite set.
theorem finite_set_image_contains_witness[T, U](s: FiniteSet[T], f: T -> U, y: U) {
    fs_image(s, f).contains(y) implies exists(x: T) {
        s.contains(x) and y = f(x)
    }
} by {
    if fs_image(s, f).contains(y) {
        finite_set_image_contains_eq(s, f, y)
        exists(x: T) {
            s.contains(x) and y = f(x)
        }
    }
}

/// The image of a member of a finite set belongs to the finite-set image.
theorem finite_set_maps_into_image[T, U](s: FiniteSet[T], f: T -> U, x: T) {
    s.contains(x) implies fs_image(s, f).contains(f(x))
} by {
    if s.contains(x) {
        finite_set_image_contains_eq(s, f, f(x))
        fs_image(s, f).contains(f(x))
    }
}

/// The image of the empty finite set is empty.
theorem finite_set_image_empty[T, U](f: T -> U) {
    fs_image(FiniteSet.empty[T], f) = FiniteSet.empty[U]
} by {
    FiniteSet.empty[T].underlying_set = Set[T].empty_set
    fs_image(FiniteSet.empty[T], f).underlying_set = set_image(Set[T].empty_set, f)
    set_image_empty(f)
    FiniteSet.empty[U].underlying_set = Set[U].empty_set
    finite_set_ext(fs_image(FiniteSet.empty[T], f), FiniteSet.empty[U])
}

/// The image of a singleton finite set is the corresponding singleton finite set.
theorem finite_set_image_singleton[T, U](a: T, f: T -> U) {
    fs_image(fs_insert(FiniteSet.empty[T], a), f) = fs_insert(FiniteSet.empty[U], f(a))
} by {
    fs_insert(FiniteSet.empty[T], a).underlying_set = FiniteSet.empty[T].underlying_set.insert(a)
    FiniteSet.empty[T].underlying_set = Set[T].empty_set
    insert_eq_union_singleton(Set[T].empty_set, a)
    Set[T].empty_set.insert(a) = Set[T].empty_set.union(Set[T].singleton(a))
    union_comm(Set[T].empty_set, Set[T].singleton(a))
    Set[T].empty_set.union(Set[T].singleton(a)) = Set[T].singleton(a).union(Set[T].empty_set)
    union_with_empty_is_self(Set[T].singleton(a))
    Set[T].empty_set.insert(a) = Set[T].singleton(a)
    fs_insert(FiniteSet.empty[T], a).underlying_set = Set[T].singleton(a)

    fs_insert(FiniteSet.empty[U], f(a)).underlying_set = FiniteSet.empty[U].underlying_set.insert(f(a))
    FiniteSet.empty[U].underlying_set = Set[U].empty_set
    insert_eq_union_singleton(Set[U].empty_set, f(a))
    Set[U].empty_set.insert(f(a)) = Set[U].empty_set.union(Set[U].singleton(f(a)))
    union_comm(Set[U].empty_set, Set[U].singleton(f(a)))
    Set[U].empty_set.union(Set[U].singleton(f(a))) = Set[U].singleton(f(a)).union(Set[U].empty_set)
    union_with_empty_is_self(Set[U].singleton(f(a)))
    Set[U].empty_set.insert(f(a)) = Set[U].singleton(f(a))
    fs_insert(FiniteSet.empty[U], f(a)).underlying_set = Set[U].singleton(f(a))

    fs_image(fs_insert(FiniteSet.empty[T], a), f).underlying_set =
        set_image(fs_insert(FiniteSet.empty[T], a).underlying_set, f)
    set_image_singleton(a, f)
    finite_set_ext(fs_image(fs_insert(FiniteSet.empty[T], a), f),
        fs_insert(FiniteSet.empty[U], f(a)))
}

/// Finite-set image is monotone with respect to finite-set subset.
theorem finite_set_image_monotone[T, U](a: FiniteSet[T], b: FiniteSet[T], f: T -> U) {
    a.subset_eq(b) implies fs_image(a, f).subset_eq(fs_image(b, f))
} by {
    if a.subset_eq(b) {
        a.underlying_set.subset(b.underlying_set)
        set_image_monotone(a.underlying_set, b.underlying_set, f)
        set_image(a.underlying_set, f).subset(set_image(b.underlying_set, f))
        fs_image(a, f).underlying_set = set_image(a.underlying_set, f)
        fs_image(b, f).underlying_set = set_image(b.underlying_set, f)
        fs_image(a, f).subset_eq(fs_image(b, f))
    }
}

/// Finite-set image distributes over finite-set union.
theorem finite_set_image_union[T, U](a: FiniteSet[T], b: FiniteSet[T], f: T -> U) {
    fs_image(fs_union(a, b), f) = fs_union(fs_image(a, f), fs_image(b, f))
} by {
    let left = fs_image(fs_union(a, b), f)
    let right = fs_union(fs_image(a, f), fs_image(b, f))
    forall(y: U) {
        if left.underlying_set.contains(y) {
            finite_set_image_contains_witness(fs_union(a, b), f, y)
            let x: T satisfy {
                fs_union(a, b).contains(x) and y = f(x)
            }
            finite_set_union_contains_eq(a, b, x)
            if a.contains(x) {
                finite_set_maps_into_image(a, f, x)
                fs_image(a, f).contains(y)
            } else {
                finite_set_maps_into_image(b, f, x)
                fs_image(b, f).contains(y)
            }
            finite_set_union_contains_eq(fs_image(a, f), fs_image(b, f), y)
            right.contains(y)
            right.underlying_set.contains(y)
        }
        if right.underlying_set.contains(y) {
            finite_set_union_contains_eq(fs_image(a, f), fs_image(b, f), y)
            if fs_image(a, f).contains(y) {
                finite_set_image_contains_witness(a, f, y)
                let x: T satisfy {
                    a.contains(x) and y = f(x)
                }
                finite_set_union_contains_eq(a, b, x)
                finite_set_maps_into_image(fs_union(a, b), f, x)
                left.contains(y)
            } else {
                finite_set_image_contains_witness(b, f, y)
                let x: T satisfy {
                    b.contains(x) and y = f(x)
                }
                finite_set_union_contains_eq(a, b, x)
                finite_set_maps_into_image(fs_union(a, b), f, x)
                left.contains(y)
            }
            left.underlying_set.contains(y)
        }
        left.underlying_set.contains(y) = right.underlying_set.contains(y)
    }
    set_ext(left.underlying_set, right.underlying_set)
    finite_set_ext(left, right)
    fs_image(fs_union(a, b), f) = fs_union(fs_image(a, f), fs_image(b, f))
}

/// The finite-set image of a composite map is the iterated finite-set image.
theorem finite_set_image_compose[A, B, C](s: FiniteSet[A], g: A -> B, f: B -> C) {
    fs_image(fs_image(s, g), f) = fs_image(s, compose(f, g))
} by {
    let image_g = fs_image(s, g)
    let left = fs_image(image_g, f)
    let right = fs_image(s, compose(f, g))
    forall(y: C) {
        if left.underlying_set.contains(y) {
            finite_set_image_contains_witness(image_g, f, y)
            let z: B satisfy {
                image_g.contains(z) and y = f(z)
            }
            finite_set_image_contains_witness(s, g, z)
            let x: A satisfy {
                s.contains(x) and z = g(x)
            }
            compose(f, g, x) = f(g(x))
            y = compose(f, g, x)
            finite_set_maps_into_image(s, compose(f, g), x)
            right.contains(compose(f, g, x))
            right.contains(y)
            right.underlying_set.contains(y)
        }
        if right.underlying_set.contains(y) {
            right.contains(y)
            finite_set_image_contains_witness(s, compose(f, g), y)
            let x: A satisfy {
                s.contains(x) and y = compose(f, g, x)
            }
            compose(f, g, x) = f(g(x))
            y = f(g(x))
            finite_set_maps_into_image(s, g, x)
            image_g.contains(g(x))
            finite_set_maps_into_image(image_g, f, g(x))
            left.contains(f(g(x)))
            left.contains(y)
            left.underlying_set.contains(y)
        }
        left.underlying_set.contains(y) = right.underlying_set.contains(y)
    }
    set_ext(left.underlying_set, right.underlying_set)
    finite_set_ext(left, right)
    fs_image(fs_image(s, g), f) = fs_image(s, compose(f, g))
}

/// The finite-set image under the identity map is the original finite set.
theorem finite_set_image_identity[T](s: FiniteSet[T]) {
    fs_image(s, identity_fn[T]) = s
} by {
    let image_set = fs_image(s, identity_fn[T]).underlying_set
    let source_set = s.underlying_set
    forall(x: T) {
        if image_set.contains(x) {
            fs_image(s, identity_fn[T]).contains(x)
            finite_set_image_contains_witness(s, identity_fn[T], x)
            let y: T satisfy {
                s.contains(y) and x = identity_fn(y)
            }
            identity_fn[T](y) = y
            x = y
            s.contains(x)
            source_set.contains(x)
        }
        if source_set.contains(x) {
            s.contains(x)
            finite_set_maps_into_image(s, identity_fn[T], x)
            fs_image(s, identity_fn[T]).contains(identity_fn[T](x))
            identity_fn[T](x) = x
            fs_image(s, identity_fn[T]).contains(x)
            image_set.contains(x)
        }
        image_set.contains(x) = source_set.contains(x)
    }
    set_ext(image_set, source_set)
    fs_image(s, identity_fn[T]).underlying_set = s.underlying_set
    finite_set_ext(fs_image(s, identity_fn[T]), s)
}

/// The predicate that an element lies in a finite set and satisfies a condition.
define finite_set_witness_predicate[T](s: FiniteSet[T], p: T -> Bool, x: T) -> Bool {
    set_witness_predicate(s.underlying_set, p, x)
}

/// A finite-set witness belongs to the finite set.
theorem finite_set_witness_contains[T](s: FiniteSet[T], p: T -> Bool, x: T) {
    finite_set_witness_predicate(s, p, x) implies s.contains(x)
} by {
    if finite_set_witness_predicate(s, p, x) {
        set_witness_contains(s.underlying_set, p, x)
        s.underlying_set.contains(x)
        s.contains(x)
    }
}

/// A finite-set witness satisfies the predicate.
theorem finite_set_witness_property[T](s: FiniteSet[T], p: T -> Bool, x: T) {
    finite_set_witness_predicate(s, p, x) implies p(x)
} by {
    if finite_set_witness_predicate(s, p, x) {
        set_witness_property(s.underlying_set, p, x)
    }
}

/// Finite-set membership together with the predicate gives a finite-set witness.
theorem finite_set_witness_intro[T](s: FiniteSet[T], p: T -> Bool, x: T) {
    s.contains(x) and p(x) implies finite_set_witness_predicate(s, p, x)
} by {
    if s.contains(x) and p(x) {
        s.underlying_set.contains(x)
        set_witness_intro(s.underlying_set, p, x)
    }
}

/// A concrete finite-set witness gives existence of a finite-set witness.
theorem finite_set_witness_exists_of_witness[T](s: FiniteSet[T], p: T -> Bool, x: T) {
    finite_set_witness_predicate(s, p, x) implies exists(y: T) {
        finite_set_witness_predicate(s, p, y)
    }
} by {
    if finite_set_witness_predicate(s, p, x) {
        exists_intro(finite_set_witness_predicate(s, p), x)
    }
}

/// A chosen element of a finite set satisfying a condition, with a specified default otherwise.
define choose_from_finite_set_or_default[T](s: FiniteSet[T], p: T -> Bool, default: T) -> T {
    choose_from_set_or_default(s.underlying_set, p, default)
}

/// The chosen finite-set element satisfies the witness predicate when such an element exists.
theorem choose_from_finite_set_or_default_spec[T](s: FiniteSet[T], p: T -> Bool, default: T) {
    exists(x: T) {
        finite_set_witness_predicate(s, p, x)
    } implies finite_set_witness_predicate(s, p, choose_from_finite_set_or_default(s, p, default))
} by {
    if exists(x: T) { finite_set_witness_predicate(s, p, x) } {
        choose_from_set_or_default_spec(s.underlying_set, p, default)
        set_witness_predicate(s.underlying_set, p, choose_from_set_or_default(s.underlying_set, p, default))
    }
}

/// The chosen finite-set element belongs to the finite set when such an element exists.
theorem choose_from_finite_set_or_default_contains[T](s: FiniteSet[T], p: T -> Bool, default: T) {
    exists(x: T) {
        finite_set_witness_predicate(s, p, x)
    } implies s.contains(choose_from_finite_set_or_default(s, p, default))
} by {
    if exists(x: T) { finite_set_witness_predicate(s, p, x) } {
        choose_from_set_or_default_contains(s.underlying_set, p, default)
        s.underlying_set.contains(choose_from_set_or_default(s.underlying_set, p, default))
        s.contains(choose_from_finite_set_or_default(s, p, default))
    }
}

/// The chosen finite-set element satisfies the condition when such an element exists.
theorem choose_from_finite_set_or_default_property[T](s: FiniteSet[T], p: T -> Bool, default: T) {
    exists(x: T) {
        finite_set_witness_predicate(s, p, x)
    } implies p(choose_from_finite_set_or_default(s, p, default))
} by {
    if exists(x: T) { finite_set_witness_predicate(s, p, x) } {
        choose_from_set_or_default_property(s.underlying_set, p, default)
    }
}

/// The default is chosen when no finite-set element satisfies the condition.
theorem choose_from_finite_set_or_default_eq_default[T](s: FiniteSet[T], p: T -> Bool, default: T) {
    not exists(x: T) {
        finite_set_witness_predicate(s, p, x)
    } implies choose_from_finite_set_or_default(s, p, default) = default
} by {
    if not exists(x: T) { finite_set_witness_predicate(s, p, x) } {
        choose_from_set_or_default_eq_default(s.underlying_set, p, default)
    }
}

/// A unique finite-set witness determines the chosen element.
theorem choose_from_finite_set_or_default_unique_eq[T](s: FiniteSet[T], p: T -> Bool, default: T, x: T) {
    exists_unique(finite_set_witness_predicate(s, p)) and finite_set_witness_predicate(s, p, x)
    implies choose_from_finite_set_or_default(s, p, default) = x
} by {
    if exists_unique(finite_set_witness_predicate(s, p)) and finite_set_witness_predicate(s, p, x) {
        exists_unique_exists(finite_set_witness_predicate(s, p))
        choose_from_finite_set_or_default_spec(s, p, default)
        finite_set_witness_predicate(s, p, choose_from_finite_set_or_default(s, p, default))
        exists_unique_eq(finite_set_witness_predicate(s, p),
            choose_from_finite_set_or_default(s, p, default), x)
    }
}

/// A finite set made from a list has cardinality at most the length of the list.
theorem finite_set_from_list_cardinality_at_most_length[T](items: List[T]) {
    fs_from_list(items).cardinality_at_most(items.length)
} by {
    list_set_cardinality_at_most_length(items)
    fs_from_list(items).underlying_set = list_set(items)
    fs_from_list(items).underlying_set.cardinality_at_most(items.length)
    fs_from_list(items).cardinality_at_most(items.length)
}

/// A finite set made from a unique list has cardinality equal to the length of the list.
theorem finite_set_from_unique_list_cardinality_is_length[T](items: List[T]) {
    items.is_unique implies fs_from_list(items).cardinality_is(items.length)
} by {
    if items.is_unique {
        unique_list_set_cardinality_is_length(items)
        fs_from_list(items).underlying_set = list_set(items)
        fs_from_list(items).underlying_set.cardinality_is(items.length)
        fs_from_list(items).cardinality_is(items.length)
    }
}

/// A finite subset of a list-backed set is extracted by filtering the list.
theorem finite_set_filter_extracts_subset[T](items: List[T], s: FiniteSet[T]) {
    s.underlying_set.subset(list_set(items)) implies
    fs_from_list(items.filter(s.underlying_set.contains)) = s
} by {
    if s.underlying_set.subset(list_set(items)) {
        list_set_filter_extracts_subset(items, s.underlying_set)
        list_set(items.filter(s.underlying_set.contains)) = s.underlying_set
        fs_from_list(items.filter(s.underlying_set.contains)).underlying_set =
            list_set(items.filter(s.underlying_set.contains))
        fs_from_list(items.filter(s.underlying_set.contains)).underlying_set = s.underlying_set
        finite_set_ext(fs_from_list(items.filter(s.underlying_set.contains)), s)
        fs_from_list(items.filter(s.underlying_set.contains)) = s
    }
}

/// A finite subset of a list-backed set has a list representation.
theorem finite_set_subset_has_list[T](items: List[T], s: FiniteSet[T]) {
    items.contains_set(s.underlying_set) implies exists(subitems: List[T]) {
        fs_from_list(subitems) = s
    }
} by {
    if items.contains_set(s.underlying_set) {
        items.contains_set(s.underlying_set) = forall(y: T) {
            s.underlying_set.contains(y) implies items.contains(y)
        }
        forall(x: T) {
            if s.underlying_set.contains(x) {
                items.contains(x)
                list_set_contains_eq(items, x)
                list_set(items).contains(x)
            }
        }
        s.underlying_set.subset(list_set(items))
        let subitems = items.filter(s.underlying_set.contains)
        finite_set_filter_extracts_subset(items, s)
        fs_from_list(subitems) = s
        exists(result: List[T]) {
            fs_from_list(result) = s
        }
    }
}

/// A finite set has a list representation with the same elements.
theorem finite_set_has_list[T](s: FiniteSet[T]) {
    exists(items: List[T]) {
        fs_from_list(items) = s
    }
} by {
    set_has_exact_containing_list(s.underlying_set)
    let items: List[T] satisfy {
        forall(x: T) {
            s.underlying_set.contains(x) = items.contains(x)
        } and items.is_unique
    }
    forall(x: T) {
        list_set_contains_eq(items, x)
        list_set(items).contains(x) = items.contains(x)
        s.underlying_set.contains(x) = list_set(items).contains(x)
    }
    set_ext(s.underlying_set, list_set(items))
    s.underlying_set = list_set(items)
    fs_from_list(items).underlying_set = list_set(items)
    fs_from_list(items).underlying_set = s.underlying_set
    finite_set_ext(fs_from_list(items), s)
    fs_from_list(items) = s
    exists(result: List[T]) {
        fs_from_list(result) = s
    }
}

/// A finite set has a unique list representation with the same elements.
theorem finite_set_has_unique_list[T](s: FiniteSet[T]) {
    exists(items: List[T]) {
        fs_from_list(items) = s and items.is_unique
    }
} by {
    set_has_exact_containing_list(s.underlying_set)
    let items: List[T] satisfy {
        forall(x: T) {
            s.underlying_set.contains(x) = items.contains(x)
        } and items.is_unique
    }
    forall(x: T) {
        list_set_contains_eq(items, x)
        list_set(items).contains(x) = items.contains(x)
        s.underlying_set.contains(x) = list_set(items).contains(x)
    }
    set_ext(s.underlying_set, list_set(items))
    s.underlying_set = list_set(items)
    fs_from_list(items).underlying_set = list_set(items)
    fs_from_list(items).underlying_set = s.underlying_set
    finite_set_ext(fs_from_list(items), s)
    fs_from_list(items) = s
    exists(result: List[T]) {
        fs_from_list(result) = s and result.is_unique
    }
}

/// A nonempty finite subset of a linear order has a least element.
theorem finite_set_nonempty_has_least[T: LinearOrder](s: FiniteSet[T]) {
    not s.is_empty implies exists(least: T) {
        s.contains(least) and forall(item: T) {
            s.contains(item) implies least <= item
        }
    }
} by {
    if not s.is_empty {
        if not exists(item: T) {
            s.contains(item)
        } {
            forall(item: T) {
                not s.contains(item)
                not s.underlying_set.contains(item)
            }
            s.underlying_set.is_empty
            s.is_empty = s.underlying_set.is_empty
            s.is_empty
            false
        }
        let witness: T satisfy {
            s.contains(witness)
        }
        finite_set_has_unique_list(s)
        let items: List[T] satisfy {
            fs_from_list(items) = s and items.is_unique
        }
        fs_from_list(items).contains(witness)
        fs_from_list(items).underlying_set = list_set(items)
        list_set(items).contains(witness)
        list_set_contains_eq(items, witness)
        items.contains(witness)
        list_contains_has_least(items, witness)
        let least: T satisfy {
            items.contains(least) and forall(item: T) {
                items.contains(item) implies least <= item
            }
        }
        list_set_contains_eq(items, least)
        list_set(items).contains(least)
        fs_from_list(items).contains(least)
        s.contains(least)
        forall(item: T) {
            if s.contains(item) {
                fs_from_list(items).contains(item)
                list_set(items).contains(item)
                list_set_contains_eq(items, item)
                items.contains(item)
                least <= item
            }
        }
        exists(result: T) {
            result = least and s.contains(result) and forall(item: T) {
                s.contains(item) implies result <= item
            }
        }
        exists(result: T) {
            s.contains(result) and forall(item: T) {
                s.contains(item) implies result <= item
            }
        }
    }
}

/// A nonempty finite subset of a linear order has a greatest element.
theorem finite_set_nonempty_has_greatest[T: LinearOrder](s: FiniteSet[T]) {
    not s.is_empty implies exists(greatest: T) {
        s.contains(greatest) and forall(item: T) {
            s.contains(item) implies item <= greatest
        }
    }
} by {
    if not s.is_empty {
        if not exists(item: T) {
            s.contains(item)
        } {
            forall(item: T) {
                not s.contains(item)
                not s.underlying_set.contains(item)
            }
            s.underlying_set.is_empty
            s.is_empty = s.underlying_set.is_empty
            s.is_empty
            false
        }
        let witness: T satisfy {
            s.contains(witness)
        }
        finite_set_has_unique_list(s)
        let items: List[T] satisfy {
            fs_from_list(items) = s and items.is_unique
        }
        fs_from_list(items).contains(witness)
        fs_from_list(items).underlying_set = list_set(items)
        list_set(items).contains(witness)
        list_set_contains_eq(items, witness)
        items.contains(witness)
        list_contains_has_greatest(items, witness)
        let greatest: T satisfy {
            items.contains(greatest) and forall(item: T) {
                items.contains(item) implies item <= greatest
            }
        }
        list_set_contains_eq(items, greatest)
        list_set(items).contains(greatest)
        fs_from_list(items).contains(greatest)
        s.contains(greatest)
        forall(item: T) {
            if s.contains(item) {
                fs_from_list(items).contains(item)
                list_set(items).contains(item)
                list_set_contains_eq(items, item)
                items.contains(item)
                item <= greatest
            }
        }
        exists(result: T) {
            result = greatest and s.contains(result) and forall(item: T) {
                s.contains(item) implies item <= result
            }
        }
        exists(result: T) {
            s.contains(result) and forall(item: T) {
                s.contains(item) implies item <= result
            }
        }
    }
}

/// A nonempty finite subset of a partial order has a maximal element.
theorem finite_set_nonempty_has_maximal[T: PartialOrder](s: FiniteSet[T]) {
    not s.is_empty implies exists(maximal: T) {
        s.contains(maximal) and forall(item: T) {
            s.contains(item) and maximal <= item implies item = maximal
        }
    }
} by {
    if not s.is_empty {
        if not exists(item: T) {
            s.contains(item)
        } {
            forall(item: T) {
                not s.contains(item)
                not s.underlying_set.contains(item)
            }
            s.underlying_set.is_empty
            s.is_empty = s.underlying_set.is_empty
            s.is_empty
            false
        }
        let witness: T satisfy {
            s.contains(witness)
        }
        finite_set_has_unique_list(s)
        let items: List[T] satisfy {
            fs_from_list(items) = s and items.is_unique
        }
        fs_from_list(items).contains(witness)
        fs_from_list(items).underlying_set = list_set(items)
        list_set(items).contains(witness)
        list_set_contains_eq(items, witness)
        items.contains(witness)
        list_contains_has_maximal(items, witness)
        let maximal: T satisfy {
            items.contains(maximal) and forall(item: T) {
                items.contains(item) and maximal <= item implies item = maximal
            }
        }
        list_set_contains_eq(items, maximal)
        list_set(items).contains(maximal)
        fs_from_list(items).contains(maximal)
        s.contains(maximal)
        forall(item: T) {
            if s.contains(item) and maximal <= item {
                fs_from_list(items).contains(item)
                list_set(items).contains(item)
                list_set_contains_eq(items, item)
                items.contains(item)
                item = maximal
            }
        }
        exists(result: T) {
            result = maximal and s.contains(result) and forall(item: T) {
                s.contains(item) and result <= item implies item = result
            }
        }
        exists(result: T) {
            s.contains(result) and forall(item: T) {
                s.contains(item) and result <= item implies item = result
            }
        }
    }
}

/// A nonempty finite subset of a partial order has a minimal element.
theorem finite_set_nonempty_has_minimal[T: PartialOrder](s: FiniteSet[T]) {
    not s.is_empty implies exists(minimal: T) {
        s.contains(minimal) and forall(item: T) {
            s.contains(item) and item <= minimal implies item = minimal
        }
    }
} by {
    if not s.is_empty {
        if not exists(item: T) {
            s.contains(item)
        } {
            forall(item: T) {
                not s.contains(item)
                not s.underlying_set.contains(item)
            }
            s.underlying_set.is_empty
            s.is_empty = s.underlying_set.is_empty
            s.is_empty
            false
        }
        let witness: T satisfy {
            s.contains(witness)
        }
        finite_set_has_unique_list(s)
        let items: List[T] satisfy {
            fs_from_list(items) = s and items.is_unique
        }
        fs_from_list(items).contains(witness)
        fs_from_list(items).underlying_set = list_set(items)
        list_set(items).contains(witness)
        list_set_contains_eq(items, witness)
        items.contains(witness)
        list_contains_has_minimal(items, witness)
        let minimal: T satisfy {
            items.contains(minimal) and forall(item: T) {
                items.contains(item) and item <= minimal implies item = minimal
            }
        }
        list_set_contains_eq(items, minimal)
        list_set(items).contains(minimal)
        fs_from_list(items).contains(minimal)
        s.contains(minimal)
        forall(item: T) {
            if s.contains(item) and item <= minimal {
                fs_from_list(items).contains(item)
                list_set(items).contains(item)
                list_set_contains_eq(items, item)
                items.contains(item)
                item = minimal
            }
        }
        exists(result: T) {
            result = minimal and s.contains(result) and forall(item: T) {
                s.contains(item) and item <= result implies item = result
            }
        }
        exists(result: T) {
            s.contains(result) and forall(item: T) {
                s.contains(item) and item <= result implies item = result
            }
        }
    }
}

/// Two least elements of a finite subset of a partial order are equal.
theorem finite_set_least_unique[T: PartialOrder](s: FiniteSet[T], a: T, b: T) {
    s.contains(a) and s.contains(b) and
    (forall(item: T) { s.contains(item) implies a <= item }) and
    (forall(item: T) { s.contains(item) implies b <= item })
    implies a = b
} by {
    if s.contains(a) and s.contains(b) and
        (forall(item: T) { s.contains(item) implies a <= item }) and
        (forall(item: T) { s.contains(item) implies b <= item }) {
        s.contains(a)
        s.contains(b)
        forall(item: T) { s.contains(item) implies a <= item }
        forall(item: T) { s.contains(item) implies b <= item }
        s.contains(b) implies a <= b
        s.contains(a) implies b <= a
        a <= b
        b <= a
        lte_antisymm(a, b)
        a = b
    }
}

/// Two greatest elements of a finite subset of a partial order are equal.
theorem finite_set_greatest_unique[T: PartialOrder](s: FiniteSet[T], a: T, b: T) {
    s.contains(a) and s.contains(b) and
    (forall(item: T) { s.contains(item) implies item <= a }) and
    (forall(item: T) { s.contains(item) implies item <= b })
    implies a = b
} by {
    if s.contains(a) and s.contains(b) and
        (forall(item: T) { s.contains(item) implies item <= a }) and
        (forall(item: T) { s.contains(item) implies item <= b }) {
        s.contains(a)
        s.contains(b)
        forall(item: T) { s.contains(item) implies item <= a }
        forall(item: T) { s.contains(item) implies item <= b }
        s.contains(b) implies b <= a
        s.contains(a) implies a <= b
        b <= a
        a <= b
        lte_antisymm(a, b)
        a = b
    }
}

/// Inclusion reverses the comparison of least elements of finite subsets of a partial order.
theorem finite_set_least_lte_of_subset[T: PartialOrder](
    s: FiniteSet[T], t: FiniteSet[T], least_s: T, least_t: T
) {
    s.subset_eq(t) and s.contains(least_s) and
    (forall(item: T) { s.contains(item) implies least_s <= item }) and
    t.contains(least_t) and
    (forall(item: T) { t.contains(item) implies least_t <= item })
    implies least_t <= least_s
} by {
    if s.subset_eq(t) and s.contains(least_s) and
        (forall(item: T) { s.contains(item) implies least_s <= item }) and
        t.contains(least_t) and
        (forall(item: T) { t.contains(item) implies least_t <= item }) {
        finite_set_subset_contains(s, t, least_s)
        t.contains(least_s)
        forall(item: T) { t.contains(item) implies least_t <= item }
        t.contains(least_s) implies least_t <= least_s
        least_t <= least_s
    }
}

/// Inclusion preserves the comparison of greatest elements of finite subsets of a partial order.
theorem finite_set_greatest_lte_of_subset[T: PartialOrder](
    s: FiniteSet[T], t: FiniteSet[T], greatest_s: T, greatest_t: T
) {
    s.subset_eq(t) and s.contains(greatest_s) and
    (forall(item: T) { s.contains(item) implies item <= greatest_s }) and
    t.contains(greatest_t) and
    (forall(item: T) { t.contains(item) implies item <= greatest_t })
    implies greatest_s <= greatest_t
} by {
    if s.subset_eq(t) and s.contains(greatest_s) and
        (forall(item: T) { s.contains(item) implies item <= greatest_s }) and
        t.contains(greatest_t) and
        (forall(item: T) { t.contains(item) implies item <= greatest_t }) {
        finite_set_subset_contains(s, t, greatest_s)
        t.contains(greatest_s)
        forall(item: T) { t.contains(item) implies item <= greatest_t }
        t.contains(greatest_s) implies greatest_s <= greatest_t
        greatest_s <= greatest_t
    }
}

/// A least element of a finite subset of a partial order lies below every greatest element.
theorem finite_set_least_lte_greatest[T: PartialOrder](
    s: FiniteSet[T], least: T, greatest: T
) {
    s.contains(least) and
    (forall(item: T) { s.contains(item) implies least <= item }) and
    s.contains(greatest) and
    (forall(item: T) { s.contains(item) implies item <= greatest })
    implies least <= greatest
} by {
    if s.contains(least) and
        (forall(item: T) { s.contains(item) implies least <= item }) and
        s.contains(greatest) and
        (forall(item: T) { s.contains(item) implies item <= greatest }) {
        forall(item: T) { s.contains(item) implies least <= item }
        s.contains(greatest) implies least <= greatest
        least <= greatest
    }
}

/// A nonempty finite subset of a linear order has comparable least and greatest elements.
theorem finite_set_nonempty_has_least_and_greatest[T: LinearOrder](s: FiniteSet[T]) {
    not s.is_empty implies exists(least: T, greatest: T) {
        s.contains(least) and
        (forall(item: T) { s.contains(item) implies least <= item }) and
        s.contains(greatest) and
        (forall(item: T) { s.contains(item) implies item <= greatest }) and
        least <= greatest
    }
} by {
    if not s.is_empty {
        finite_set_nonempty_has_least(s)
        let least: T satisfy {
            s.contains(least) and forall(item: T) {
                s.contains(item) implies least <= item
            }
        }
        finite_set_nonempty_has_greatest(s)
        let greatest: T satisfy {
            s.contains(greatest) and forall(item: T) {
                s.contains(item) implies item <= greatest
            }
        }
        finite_set_least_lte_greatest(s, least, greatest)
        exists(result_least: T, result_greatest: T) {
            result_least = least and result_greatest = greatest and
            s.contains(result_least) and
            (forall(item: T) { s.contains(item) implies result_least <= item }) and
            s.contains(result_greatest) and
            (forall(item: T) { s.contains(item) implies item <= result_greatest }) and
            result_least <= result_greatest
        }
        exists(result_least: T, result_greatest: T) {
            s.contains(result_least) and
            (forall(item: T) { s.contains(item) implies result_least <= item }) and
            s.contains(result_greatest) and
            (forall(item: T) { s.contains(item) implies item <= result_greatest }) and
            result_least <= result_greatest
        }
    }
}

/// The least element of a finite-set union lies below the least element of its left operand.
theorem finite_set_union_least_lte_left[T: PartialOrder](
    s: FiniteSet[T], t: FiniteSet[T], least_s: T, least_union: T
) {
    s.contains(least_s) and
    (forall(item: T) { s.contains(item) implies least_s <= item }) and
    fs_union(s, t).contains(least_union) and
    (forall(item: T) { fs_union(s, t).contains(item) implies least_union <= item })
    implies least_union <= least_s
} by {
    if s.contains(least_s) and
        (forall(item: T) { s.contains(item) implies least_s <= item }) and
        fs_union(s, t).contains(least_union) and
        (forall(item: T) { fs_union(s, t).contains(item) implies least_union <= item }) {
        s.contains(least_s)
        finite_set_subset_union_left(s, t)
        finite_set_subset_contains(s, fs_union(s, t), least_s)
        fs_union(s, t).contains(least_s)
        forall(item: T) { fs_union(s, t).contains(item) implies least_union <= item }
        fs_union(s, t).contains(least_s) implies least_union <= least_s
        least_union <= least_s
    }
}

/// The least element of a finite-set union lies below the least element of its right operand.
theorem finite_set_union_least_lte_right[T: PartialOrder](
    s: FiniteSet[T], t: FiniteSet[T], least_t: T, least_union: T
) {
    t.contains(least_t) and
    (forall(item: T) { t.contains(item) implies least_t <= item }) and
    fs_union(s, t).contains(least_union) and
    (forall(item: T) { fs_union(s, t).contains(item) implies least_union <= item })
    implies least_union <= least_t
} by {
    if t.contains(least_t) and
        (forall(item: T) { t.contains(item) implies least_t <= item }) and
        fs_union(s, t).contains(least_union) and
        (forall(item: T) { fs_union(s, t).contains(item) implies least_union <= item }) {
        t.contains(least_t)
        finite_set_subset_union_right(s, t)
        finite_set_subset_contains(t, fs_union(s, t), least_t)
        fs_union(s, t).contains(least_t)
        forall(item: T) { fs_union(s, t).contains(item) implies least_union <= item }
        fs_union(s, t).contains(least_t) implies least_union <= least_t
        least_union <= least_t
    }
}

/// The greatest element of a finite-set union lies above the greatest element of its left operand.
theorem finite_set_union_greatest_gte_left[T: PartialOrder](
    s: FiniteSet[T], t: FiniteSet[T], greatest_s: T, greatest_union: T
) {
    s.contains(greatest_s) and
    (forall(item: T) { s.contains(item) implies item <= greatest_s }) and
    fs_union(s, t).contains(greatest_union) and
    (forall(item: T) { fs_union(s, t).contains(item) implies item <= greatest_union })
    implies greatest_s <= greatest_union
} by {
    if s.contains(greatest_s) and
        (forall(item: T) { s.contains(item) implies item <= greatest_s }) and
        fs_union(s, t).contains(greatest_union) and
        (forall(item: T) { fs_union(s, t).contains(item) implies item <= greatest_union }) {
        s.contains(greatest_s)
        finite_set_subset_union_left(s, t)
        finite_set_subset_contains(s, fs_union(s, t), greatest_s)
        fs_union(s, t).contains(greatest_s)
        forall(item: T) { fs_union(s, t).contains(item) implies item <= greatest_union }
        fs_union(s, t).contains(greatest_s) implies greatest_s <= greatest_union
        greatest_s <= greatest_union
    }
}

/// The greatest element of a finite-set union lies above the greatest element of its right operand.
theorem finite_set_union_greatest_gte_right[T: PartialOrder](
    s: FiniteSet[T], t: FiniteSet[T], greatest_t: T, greatest_union: T
) {
    t.contains(greatest_t) and
    (forall(item: T) { t.contains(item) implies item <= greatest_t }) and
    fs_union(s, t).contains(greatest_union) and
    (forall(item: T) { fs_union(s, t).contains(item) implies item <= greatest_union })
    implies greatest_t <= greatest_union
} by {
    if t.contains(greatest_t) and
        (forall(item: T) { t.contains(item) implies item <= greatest_t }) and
        fs_union(s, t).contains(greatest_union) and
        (forall(item: T) { fs_union(s, t).contains(item) implies item <= greatest_union }) {
        t.contains(greatest_t)
        finite_set_subset_union_right(s, t)
        finite_set_subset_contains(t, fs_union(s, t), greatest_t)
        fs_union(s, t).contains(greatest_t)
        forall(item: T) { fs_union(s, t).contains(item) implies item <= greatest_union }
        fs_union(s, t).contains(greatest_t) implies greatest_t <= greatest_union
        greatest_t <= greatest_union
    }
}

/// The minimum of two least elements belongs to the union of their finite sets.
theorem finite_set_union_contains_min_of_leasts[T: LinearOrder](
    s: FiniteSet[T], t: FiniteSet[T], least_s: T, least_t: T
) {
    s.contains(least_s) and t.contains(least_t)
    implies fs_union(s, t).contains(least_s.min(least_t))
} by {
    if s.contains(least_s) and t.contains(least_t) {
        min_is_one(least_s, least_t)
        if least_s.min(least_t) = least_s {
            finite_set_union_contains_eq(s, t, least_s)
            fs_union(s, t).contains(least_s)
            fs_union(s, t).contains(least_s.min(least_t))
        } else {
            least_s.min(least_t) = least_t
            finite_set_union_contains_eq(s, t, least_t)
            fs_union(s, t).contains(least_t)
            fs_union(s, t).contains(least_s.min(least_t))
        }
    }
}

/// The minimum of lower bounds for two finite sets is a lower bound for their union.
theorem finite_set_union_min_is_lower_bound[T: LinearOrder](
    s: FiniteSet[T], t: FiniteSet[T], least_s: T, least_t: T
) {
    (forall(item: T) { s.contains(item) implies least_s <= item }) and
    (forall(item: T) { t.contains(item) implies least_t <= item })
    implies forall(item: T) {
        fs_union(s, t).contains(item) implies least_s.min(least_t) <= item
    }
} by {
    if (forall(item: T) { s.contains(item) implies least_s <= item }) and
        (forall(item: T) { t.contains(item) implies least_t <= item }) {
        forall(item: T) {
            if fs_union(s, t).contains(item) {
                finite_set_union_contains_eq(s, t, item)
                if s.contains(item) {
                    min_lte_left(least_s, least_t)
                    s.contains(item) implies least_s <= item
                    lte_trans(least_s.min(least_t), least_s, item)
                    least_s.min(least_t) <= item
                }
                if t.contains(item) {
                    min_lte_right(least_s, least_t)
                    t.contains(item) implies least_t <= item
                    lte_trans(least_s.min(least_t), least_t, item)
                    least_s.min(least_t) <= item
                }
            }
        }
        forall(item: T) {
            fs_union(s, t).contains(item) implies least_s.min(least_t) <= item
        }
    }
}

/// A least element of a finite-set union is the minimum of the operands' least elements.
theorem finite_set_union_least_eq_min[T: LinearOrder](
    s: FiniteSet[T], t: FiniteSet[T], least_s: T, least_t: T, least_union: T
) {
    s.contains(least_s) and
    (forall(item: T) { s.contains(item) implies least_s <= item }) and
    t.contains(least_t) and
    (forall(item: T) { t.contains(item) implies least_t <= item }) and
    fs_union(s, t).contains(least_union) and
    (forall(item: T) { fs_union(s, t).contains(item) implies least_union <= item })
    implies least_union = least_s.min(least_t)
} by {
    if s.contains(least_s) and
        (forall(item: T) { s.contains(item) implies least_s <= item }) and
        t.contains(least_t) and
        (forall(item: T) { t.contains(item) implies least_t <= item }) and
        fs_union(s, t).contains(least_union) and
        (forall(item: T) { fs_union(s, t).contains(item) implies least_union <= item }) {
        finite_set_union_contains_min_of_leasts(s, t, least_s, least_t)
        finite_set_union_min_is_lower_bound(s, t, least_s, least_t)
        fs_union(s, t).contains(least_s.min(least_t))
        forall(item: T) { fs_union(s, t).contains(item) implies least_union <= item }
        fs_union(s, t).contains(least_s.min(least_t)) implies least_union <= least_s.min(least_t)
        least_union <= least_s.min(least_t)
        forall(item: T) {
            fs_union(s, t).contains(item) implies least_s.min(least_t) <= item
        }
        fs_union(s, t).contains(least_union) implies least_s.min(least_t) <= least_union
        least_s.min(least_t) <= least_union
        lte_antisymm(least_union, least_s.min(least_t))
        least_union = least_s.min(least_t)
    }
}

/// The maximum of two greatest elements belongs to the union of their finite sets.
theorem finite_set_union_contains_max_of_greatests[T: LinearOrder](
    s: FiniteSet[T], t: FiniteSet[T], greatest_s: T, greatest_t: T
) {
    s.contains(greatest_s) and t.contains(greatest_t)
    implies fs_union(s, t).contains(greatest_s.max(greatest_t))
} by {
    if s.contains(greatest_s) and t.contains(greatest_t) {
        max_is_one(greatest_s, greatest_t)
        if greatest_s.max(greatest_t) = greatest_s {
            finite_set_union_contains_eq(s, t, greatest_s)
            fs_union(s, t).contains(greatest_s)
            fs_union(s, t).contains(greatest_s.max(greatest_t))
        } else {
            greatest_s.max(greatest_t) = greatest_t
            finite_set_union_contains_eq(s, t, greatest_t)
            fs_union(s, t).contains(greatest_t)
            fs_union(s, t).contains(greatest_s.max(greatest_t))
        }
    }
}

/// The maximum of upper bounds for two finite sets is an upper bound for their union.
theorem finite_set_union_max_is_upper_bound[T: LinearOrder](
    s: FiniteSet[T], t: FiniteSet[T], greatest_s: T, greatest_t: T
) {
    (forall(item: T) { s.contains(item) implies item <= greatest_s }) and
    (forall(item: T) { t.contains(item) implies item <= greatest_t })
    implies forall(item: T) {
        fs_union(s, t).contains(item) implies item <= greatest_s.max(greatest_t)
    }
} by {
    if (forall(item: T) { s.contains(item) implies item <= greatest_s }) and
        (forall(item: T) { t.contains(item) implies item <= greatest_t }) {
        forall(item: T) {
            if fs_union(s, t).contains(item) {
                finite_set_union_contains_eq(s, t, item)
                if s.contains(item) {
                    s.contains(item) implies item <= greatest_s
                    lte_max_left(greatest_s, greatest_t)
                    lte_trans(item, greatest_s, greatest_s.max(greatest_t))
                    item <= greatest_s.max(greatest_t)
                }
                if t.contains(item) {
                    t.contains(item) implies item <= greatest_t
                    lte_max_right(greatest_s, greatest_t)
                    lte_trans(item, greatest_t, greatest_s.max(greatest_t))
                    item <= greatest_s.max(greatest_t)
                }
            }
        }
        forall(item: T) {
            fs_union(s, t).contains(item) implies item <= greatest_s.max(greatest_t)
        }
    }
}

/// A greatest element of a finite-set union is the maximum of the operands' greatest elements.
theorem finite_set_union_greatest_eq_max[T: LinearOrder](
    s: FiniteSet[T], t: FiniteSet[T], greatest_s: T, greatest_t: T, greatest_union: T
) {
    s.contains(greatest_s) and
    (forall(item: T) { s.contains(item) implies item <= greatest_s }) and
    t.contains(greatest_t) and
    (forall(item: T) { t.contains(item) implies item <= greatest_t }) and
    fs_union(s, t).contains(greatest_union) and
    (forall(item: T) { fs_union(s, t).contains(item) implies item <= greatest_union })
    implies greatest_union = greatest_s.max(greatest_t)
} by {
    if s.contains(greatest_s) and
        (forall(item: T) { s.contains(item) implies item <= greatest_s }) and
        t.contains(greatest_t) and
        (forall(item: T) { t.contains(item) implies item <= greatest_t }) and
        fs_union(s, t).contains(greatest_union) and
        (forall(item: T) { fs_union(s, t).contains(item) implies item <= greatest_union }) {
        finite_set_union_contains_max_of_greatests(s, t, greatest_s, greatest_t)
        finite_set_union_max_is_upper_bound(s, t, greatest_s, greatest_t)
        fs_union(s, t).contains(greatest_s.max(greatest_t))
        forall(item: T) { fs_union(s, t).contains(item) implies item <= greatest_union }
        fs_union(s, t).contains(greatest_s.max(greatest_t)) implies greatest_s.max(greatest_t) <= greatest_union
        greatest_s.max(greatest_t) <= greatest_union
        forall(item: T) {
            fs_union(s, t).contains(item) implies item <= greatest_s.max(greatest_t)
        }
        fs_union(s, t).contains(greatest_union) implies greatest_union <= greatest_s.max(greatest_t)
        greatest_union <= greatest_s.max(greatest_t)
        lte_antisymm(greatest_union, greatest_s.max(greatest_t))
        greatest_union = greatest_s.max(greatest_t)
    }
}

/// Every member equals an element which is both least and greatest.
theorem finite_set_member_eq_of_least_and_greatest[T: PartialOrder](
    s: FiniteSet[T], extremum: T, item: T
) {
    s.contains(extremum) and
    (forall(member: T) { s.contains(member) implies extremum <= member }) and
    (forall(member: T) { s.contains(member) implies member <= extremum }) and
    s.contains(item)
    implies item = extremum
} by {
    if s.contains(extremum) and
        (forall(member: T) { s.contains(member) implies extremum <= member }) and
        (forall(member: T) { s.contains(member) implies member <= extremum }) and
        s.contains(item) {
        forall(x: T) { s.contains(x) implies extremum <= x }
        forall(x: T) { s.contains(x) implies x <= extremum }
        s.contains(item) implies extremum <= item
        s.contains(item) implies item <= extremum
        extremum <= item
        item <= extremum
        lte_antisymm(item, extremum)
        item = extremum
    }
}

/// A singleton finite set is contained in every finite set containing its element.
theorem finite_set_singleton_subset_of_contains[T](s: FiniteSet[T], item: T) {
    s.contains(item) implies fs_insert(FiniteSet.empty[T], item).subset_eq(s)
} by {
    if s.contains(item) {
        forall(x: T) {
            finite_set_singleton_contains_eq(item, x)
            if fs_insert(FiniteSet.empty[T], item).contains(x) {
                x = item
                s.contains(item)
                s.contains(x)
            }
            fs_insert(FiniteSet.empty[T], item).underlying_set.contains(x) implies s.underlying_set.contains(x)
        }
        fs_insert(FiniteSet.empty[T], item).underlying_set.subset(s.underlying_set)
        fs_insert(FiniteSet.empty[T], item).subset_eq(s)
    }
}

/// A finite subset with the same least and greatest element is contained in its singleton.
theorem finite_set_subset_singleton_of_least_and_greatest[T: PartialOrder](
    s: FiniteSet[T], extremum: T
) {
    s.contains(extremum) and
    (forall(item: T) { s.contains(item) implies extremum <= item }) and
    (forall(item: T) { s.contains(item) implies item <= extremum })
    implies s.subset_eq(fs_insert(FiniteSet.empty[T], extremum))
} by {
    if s.contains(extremum) and
        (forall(item: T) { s.contains(item) implies extremum <= item }) and
        (forall(item: T) { s.contains(item) implies item <= extremum }) {
        forall(x: T) {
            if s.contains(x) {
                finite_set_member_eq_of_least_and_greatest(s, extremum, x)
                x = extremum
                finite_set_singleton_contains_eq(extremum, x)
                fs_insert(FiniteSet.empty[T], extremum).contains(x)
            }
            s.underlying_set.contains(x) implies fs_insert(FiniteSet.empty[T], extremum).underlying_set.contains(x)
        }
        s.underlying_set.subset(fs_insert(FiniteSet.empty[T], extremum).underlying_set)
        s.subset_eq(fs_insert(FiniteSet.empty[T], extremum))
    }
}

/// A finite subset with the same least and greatest element is a singleton.
theorem finite_set_eq_singleton_of_least_and_greatest[T: PartialOrder](
    s: FiniteSet[T], extremum: T
) {
    s.contains(extremum) and
    (forall(item: T) { s.contains(item) implies extremum <= item }) and
    (forall(item: T) { s.contains(item) implies item <= extremum })
    implies s = fs_insert(FiniteSet.empty[T], extremum)
} by {
    if s.contains(extremum) and
        (forall(item: T) { s.contains(item) implies extremum <= item }) and
        (forall(item: T) { s.contains(item) implies item <= extremum }) {
        finite_set_subset_singleton_of_least_and_greatest(s, extremum)
        finite_set_singleton_subset_of_contains(s, extremum)
        finite_set_subset_antisymm(s, fs_insert(FiniteSet.empty[T], extremum))
        s = fs_insert(FiniteSet.empty[T], extremum)
    }
}

/// The element of a singleton finite subset is a lower bound.
theorem finite_set_singleton_element_is_lower_bound[T: PartialOrder](item: T) {
    forall(member: T) {
        fs_insert(FiniteSet.empty[T], item).contains(member) implies item <= member
    }
} by {
    forall(member: T) {
        if fs_insert(FiniteSet.empty[T], item).contains(member) {
            finite_set_singleton_contains_eq(item, member)
            member = item
            lte_refl(item)
            item <= member
        }
    }
}

/// The element of a singleton finite subset is an upper bound.
theorem finite_set_singleton_element_is_upper_bound[T: PartialOrder](item: T) {
    forall(member: T) {
        fs_insert(FiniteSet.empty[T], item).contains(member) implies member <= item
    }
} by {
    forall(member: T) {
        if fs_insert(FiniteSet.empty[T], item).contains(member) {
            finite_set_singleton_contains_eq(item, member)
            member = item
            lte_refl(item)
            member <= item
        }
    }
}

/// A singleton finite set has cardinality one.
theorem finite_set_singleton_cardinality_is_one[T](item: T) {
    fs_insert(FiniteSet.empty[T], item).cardinality_is(Nat.1)
} by {
    forall(x: T) {
        finite_set_singleton_contains_eq(item, x)
        singleton_contains_eq(item, x)
        fs_insert(FiniteSet.empty[T], item).underlying_set.contains(x) =
            Set[T].singleton(item).contains(x)
    }
    set_ext(fs_insert(FiniteSet.empty[T], item).underlying_set, Set[T].singleton(item))
    fs_insert(FiniteSet.empty[T], item).underlying_set = Set[T].singleton(item)
    singleton_set_cardinality_is_one(item)
    fs_insert(FiniteSet.empty[T], item).cardinality_is(Nat.1)
}

/// A finite set with exact cardinality is represented by a unique list of that length.
theorem finite_set_cardinality_has_exact_unique_list[T](s: FiniteSet[T], n: Nat) {
    s.cardinality_is(n) implies exists(items: List[T]) {
        fs_from_list(items) = s and items.is_unique and items.length = n
    }
} by {
    if s.cardinality_is(n) {
        s.underlying_set.cardinality_is(n)
        set_cardinality_has_exact_unique_list(s.underlying_set, n)
        let items: List[T] satisfy {
            list_set(items) = s.underlying_set and items.is_unique and items.length = n
        }

        fs_from_list(items).underlying_set = list_set(items)
        fs_from_list(items).underlying_set = s.underlying_set
        finite_set_ext(fs_from_list(items), s)
        fs_from_list(items) = s
        exists(result: List[T]) {
            fs_from_list(result) = s and result.is_unique and result.length = n
        }
    }
}

/// A finite set of cardinality one is the singleton of any element it contains.
theorem finite_set_eq_singleton_of_cardinality_one_contains[T](s: FiniteSet[T], item: T) {
    s.cardinality_is(Nat.1) and s.contains(item) implies
        s = fs_insert(FiniteSet.empty[T], item)
} by {
    if s.cardinality_is(Nat.1) and s.contains(item) {
        finite_set_cardinality_has_exact_unique_list(s, Nat.1)
        let items: List[T] satisfy {
            fs_from_list(items) = s and items.is_unique and items.length = Nat.1
        }
        match items {
            List.nil {
                items.length = Nat.0
                Nat.0 != Nat.1
                false
            }
            List.cons(head, tail) {
                items.length = tail.length.suc
                Nat.1 = Nat.0.suc
                tail.length = Nat.0
                match tail {
                    List.nil {
                        tail = List.nil[T]
                        items = List.singleton(head)
                        finite_set_eq_underlying_contains_at(fs_from_list(items), s, item)
                        fs_from_list(items).underlying_set = list_set(items)
                        list_set_contains_eq(items, item)
                        items.contains(item)
                        if head != item {
                            items.contains(item) = List.nil[T].contains(item)
                            not List.nil[T].contains(item)
                            false
                        }
                        head = item
                        forall(x: T) {
                            finite_set_eq_underlying_contains_at(fs_from_list(items), s, x)
                            fs_from_list(items).underlying_set = list_set(items)
                            list_set_contains_eq(items, x)
                            finite_set_singleton_contains_eq(item, x)
                            if x = item {
                                items.contains(x)
                                s.contains(x)
                                fs_insert(FiniteSet.empty[T], item).contains(x)
                                s.underlying_set.contains(x) =
                                    fs_insert(FiniteSet.empty[T], item).underlying_set.contains(x)
                            } else {
                                head != x
                                items.contains(x) = List.nil[T].contains(x)
                                not List.nil[T].contains(x)
                                not s.contains(x)
                                not fs_insert(FiniteSet.empty[T], item).contains(x)
                                eq_false_intro(s.underlying_set.contains(x))
                                eq_false_intro(
                                    fs_insert(FiniteSet.empty[T], item).underlying_set.contains(x))
                                s.underlying_set.contains(x) = false
                                fs_insert(FiniteSet.empty[T], item).underlying_set.contains(x) = false
                                s.underlying_set.contains(x) =
                                    fs_insert(FiniteSet.empty[T], item).underlying_set.contains(x)
                            }
                        }
                        finite_set_ext_contains(s, fs_insert(FiniteSet.empty[T], item))
                        s = fs_insert(FiniteSet.empty[T], item)
                    }
                    List.cons(next, rest) {
                        tail.length = rest.length.suc
                        rest.length.suc != Nat.0
                        false
                    }
                }
            }
        }
    }
}

/// Every element of a cardinality-one finite subset is a lower bound.
theorem finite_set_cardinality_one_element_is_lower_bound[T: PartialOrder](
    s: FiniteSet[T], item: T
) {
    s.cardinality_is(Nat.1) and s.contains(item) implies forall(member: T) {
        s.contains(member) implies item <= member
    }
} by {
    if s.cardinality_is(Nat.1) and s.contains(item) {
        finite_set_eq_singleton_of_cardinality_one_contains(s, item)
        forall(member: T) {
            if s.contains(member) {
                finite_set_eq_underlying_contains_at(
                    s, fs_insert(FiniteSet.empty[T], item), member)
                s.underlying_set.contains(member) =
                    fs_insert(FiniteSet.empty[T], item).underlying_set.contains(member)
                fs_insert(FiniteSet.empty[T], item).contains(member)
                finite_set_singleton_contains_eq(item, member)
                member = item
                lte_refl(item)
                item <= member
            }
        }
    }
}

/// Every element of a cardinality-one finite subset is an upper bound.
theorem finite_set_cardinality_one_element_is_upper_bound[T: PartialOrder](
    s: FiniteSet[T], item: T
) {
    s.cardinality_is(Nat.1) and s.contains(item) implies forall(member: T) {
        s.contains(member) implies member <= item
    }
} by {
    if s.cardinality_is(Nat.1) and s.contains(item) {
        finite_set_eq_singleton_of_cardinality_one_contains(s, item)
        forall(member: T) {
            if s.contains(member) {
                finite_set_eq_underlying_contains_at(
                    s, fs_insert(FiniteSet.empty[T], item), member)
                s.underlying_set.contains(member) =
                    fs_insert(FiniteSet.empty[T], item).underlying_set.contains(member)
                fs_insert(FiniteSet.empty[T], item).contains(member)
                finite_set_singleton_contains_eq(item, member)
                member = item
                lte_refl(item)
                member <= item
            }
        }
    }
}

/// A finite set containing two distinct elements has cardinality at least two.
theorem finite_set_cardinality_at_least_two_of_contains_ne[T](
    s: FiniteSet[T], a: T, b: T, n: Nat
) {
    s.cardinality_is(n) and s.contains(a) and s.contains(b) and a != b implies Nat.2 <= n
} by {
    if s.cardinality_is(n) and s.contains(a) and s.contains(b) and a != b {
        finite_set_cardinality_has_exact_unique_list(s, n)
        let items: List[T] satisfy {
            fs_from_list(items) = s and items.is_unique and items.length = n
        }
        finite_set_eq_underlying_contains_at(fs_from_list(items), s, a)
        finite_set_eq_underlying_contains_at(fs_from_list(items), s, b)
        fs_from_list(items).underlying_set = list_set(items)
        list_set_contains_eq(items, a)
        list_set_contains_eq(items, b)
        items.contains(a)
        items.contains(b)
        match items {
            List.nil {
                not items.contains(a)
                false
            }
            List.cons(head, tail) {
                match tail {
                    List.nil {
                        tail = List.nil[T]
                        items = List.singleton(head)
                        if head != a {
                            items.contains(a) = List.nil[T].contains(a)
                            not List.nil[T].contains(a)
                            false
                        }
                        head = a
                        if head != b {
                            items.contains(b) = List.nil[T].contains(b)
                            not List.nil[T].contains(b)
                            false
                        }
                        head = b
                        a = b
                        false
                    }
                    List.cons(next, rest) {
                        tail = List.cons(next, rest)
                        items = List.cons(head, List.cons(next, rest))
                        tail.length = rest.length.suc
                        items.length = tail.length.suc
                        items.length = rest.length.suc.suc
                        Nat.2 = Nat.0.suc.suc
                        Nat.0 <= rest.length
                        lte_suc_suc(Nat.0, rest.length)
                        Nat.0.suc <= rest.length.suc
                        lte_suc_suc(Nat.0.suc, rest.length.suc)
                        Nat.0.suc.suc <= rest.length.suc.suc
                        Nat.2 <= items.length
                        Nat.2 <= n
                    }
                }
            }
        }
    }
}

/// Distinct least and greatest elements force cardinality at least two.
theorem finite_set_distinct_extrema_cardinality_at_least_two[T: PartialOrder](
    s: FiniteSet[T], least: T, greatest: T, n: Nat
) {
    s.cardinality_is(n) and
    s.contains(least) and
    (forall(item: T) { s.contains(item) implies least <= item }) and
    s.contains(greatest) and
    (forall(item: T) { s.contains(item) implies item <= greatest }) and
    least != greatest
    implies Nat.2 <= n
} by {
    finite_set_cardinality_at_least_two_of_contains_ne(s, least, greatest, n)
}

/// The image of a cardinality-bounded finite set has the same cardinality bound.
theorem finite_set_image_cardinality_at_most[T, U](s: FiniteSet[T], f: T -> U, n: Nat) {
    s.cardinality_at_most(n) implies fs_image(s, f).cardinality_at_most(n)
} by {
    if s.cardinality_at_most(n) {
        s.underlying_set.cardinality_at_most(n)
        set_image_cardinality_at_most(s.underlying_set, f, n)
        fs_image(s, f).underlying_set = set_image(s.underlying_set, f)
        fs_image(s, f).underlying_set.cardinality_at_most(n)
        fs_image(s, f).cardinality_at_most(n)
    }
}

/// The image of a finite set under an injective map has the same exact cardinality.
theorem finite_set_image_cardinality_is_of_injective[T, U](s: FiniteSet[T], f: T -> U, n: Nat) {
    is_injective_fn(f) and s.cardinality_is(n) implies fs_image(s, f).cardinality_is(n)
} by {
    if is_injective_fn(f) and s.cardinality_is(n) {
        s.underlying_set.cardinality_is(n)
        set_image_cardinality_is_of_injective(s.underlying_set, f, n)
        fs_image(s, f).underlying_set = set_image(s.underlying_set, f)
        fs_image(s, f).underlying_set.cardinality_is(n)
        fs_image(s, f).cardinality_is(n)
    }
}

/// The image of a finite set under a bijection has the same exact cardinality.
theorem finite_set_image_cardinality_is_of_bijection[T, U](s: FiniteSet[T], f: T -> U, n: Nat) {
    is_bijection_fn(f) and s.cardinality_is(n) implies fs_image(s, f).cardinality_is(n)
} by {
    if is_bijection_fn(f) and s.cardinality_is(n) {
        bijection_fn_is_injective(f)
        is_injective_fn(f)
        finite_set_image_cardinality_is_of_injective(s, f, n)
        fs_image(s, f).cardinality_is(n)
    }
}

/// Membership in a finite-set image gives membership of the chosen inverse.
theorem finite_set_image_contains_inverse_of_bijection[T: Inhabited, U](s: FiniteSet[T], f: T -> U, y: U) {
    is_bijection_fn(f) and fs_image(s, f).contains(y) implies s.contains(inverse_fn(f, y))
} by {
    if is_bijection_fn(f) and fs_image(s, f).contains(y) {
        fs_image(s, f).underlying_set = set_image(s.underlying_set, f)
        set_image(s.underlying_set, f).contains(y)
        set_image_contains_inverse_of_bijection(s.underlying_set, f, y)
        s.underlying_set.contains(inverse_fn(f, y))
        s.contains(inverse_fn(f, y))
    }
}

/// Membership of the chosen inverse gives membership in a finite-set image.
theorem finite_set_inverse_contains_imp_image_contains[T: Inhabited, U](s: FiniteSet[T], f: T -> U, y: U) {
    is_bijection_fn(f) and s.contains(inverse_fn(f, y)) implies fs_image(s, f).contains(y)
} by {
    if is_bijection_fn(f) and s.contains(inverse_fn(f, y)) {
        fs_image(s, f).underlying_set = set_image(s.underlying_set, f)
        s.underlying_set.contains(inverse_fn(f, y))
        set_inverse_contains_imp_image_contains(s.underlying_set, f, y)
        set_image(s.underlying_set, f).contains(y)
        fs_image(s, f).contains(y)
    }
}

/// Membership in a finite-set image is membership of the chosen inverse.
theorem finite_set_image_contains_iff_inverse_of_bijection[T: Inhabited, U](s: FiniteSet[T], f: T -> U, y: U) {
    is_bijection_fn(f) implies fs_image(s, f).contains(y) = s.contains(inverse_fn(f, y))
} by {
    if is_bijection_fn(f) {
        if fs_image(s, f).contains(y) {
            finite_set_image_contains_inverse_of_bijection(s, f, y)
            s.contains(inverse_fn(f, y))
        }
        if s.contains(inverse_fn(f, y)) {
            finite_set_inverse_contains_imp_image_contains(s, f, y)
            fs_image(s, f).contains(y)
        }
        fs_image(s, f).contains(y) = s.contains(inverse_fn(f, y))
    }
}

/// A map from a finite set into a shorter containing list has a collision.
theorem finite_set_pigeonhole_into_list[T, U](s: FiniteSet[T], targets: List[U], f: T -> U, n: Nat) {
    s.cardinality_is(n) and targets.length < n and
    forall(x: T) {
        s.contains(x) implies targets.contains(f(x))
    } implies exists(x: T, y: T) {
        x != y and f(x) = f(y)
    }
} by {
    finite_set_cardinality_has_exact_unique_list(s, n)
    let items: List[T] satisfy {
        fs_from_list(items) = s and items.is_unique and items.length = n
    }
    items.length > targets.length

    forall(x: T) {
        if items.contains(x) {
            list_set_contains_eq(items, x)
            fs_from_list(items).underlying_set = list_set(items)
            fs_from_list(items).contains(x)
            s.contains(x)
            targets.contains(f(x))
        }
    }
    pigeonhole_map_into_list(items, targets, f)
}

/// A map from a finite set into a smaller finite set has a collision.
theorem finite_set_pigeonhole_into_finite_set[T, U](s: FiniteSet[T], t: FiniteSet[U],
    f: T -> U, n_s: Nat, n_t: Nat) {
    s.cardinality_is(n_s) and t.cardinality_is(n_t) and n_t < n_s and
    forall(x: T) {
        s.contains(x) implies t.contains(f(x))
    } implies exists(x: T, y: T) {
        x != y and f(x) = f(y)
    }
} by {
    finite_set_cardinality_has_exact_unique_list(t, n_t)
    let targets: List[U] satisfy {
        fs_from_list(targets) = t and targets.is_unique and targets.length = n_t
    }
    targets.length < n_s

    forall(x: T) {
        if s.contains(x) {
            t.contains(f(x))
            fs_from_list(targets).contains(f(x))
            fs_from_list(targets).underlying_set = list_set(targets)
            list_set(targets).contains(f(x))
            list_set_contains_eq(targets, f(x))
            targets.contains(f(x))
        }
    }
    finite_set_pigeonhole_into_list(s, targets, f, n_s)
}

/// Exact cardinality of a finite set is unique.
theorem finite_set_cardinality_is_well_defined[T](s: FiniteSet[T], n1: Nat, n2: Nat) {
    s.cardinality_is(n1) and s.cardinality_is(n2) implies n1 = n2
} by {
    if s.cardinality_is(n1) and s.cardinality_is(n2) {
        s.underlying_set.cardinality_is(n1)
        s.underlying_set.cardinality_is(n2)
        cardinality_is_well_defined(s.underlying_set, n1, n2)
        n1 = n2
    }
}

/// Every finite set has an exact cardinality.
theorem finite_set_cardinality_exists[T](s: FiniteSet[T]) {
    exists(n: Nat) {
        s.cardinality_is(n)
    }
} by {
    s.underlying_set.is_finite
    cardinality_always_exists(s.underlying_set)
    let n: Nat satisfy {
        s.underlying_set.cardinality_is(n)
    }
    s.cardinality_is(n)
    exists(m: Nat) {
        m = n and s.cardinality_is(m)
    }
}

/// Exact cardinality gives the corresponding upper cardinality bound.
theorem finite_set_cardinality_is_smallest_cardinality[T](s: FiniteSet[T], n: Nat) {
    s.cardinality_is(n) implies s.cardinality_at_most(n)
} by {
    if s.cardinality_is(n) {
        s.underlying_set.cardinality_is(n)
        cardinality_is_smallest_cardinality(s.underlying_set, n)
        s.underlying_set.cardinality_at_most(n)
        s.cardinality_at_most(n)
    }
}

/// Exact cardinality implies finiteness of the underlying set.
theorem finite_set_cardinality_is_implies_underlying_is_finite[T](s: FiniteSet[T], n: Nat) {
    s.cardinality_is(n) implies s.underlying_set.is_finite
} by {
    if s.cardinality_is(n) {
        s.underlying_set.cardinality_is(n)
        cardinality_is_implies_is_finite(s.underlying_set, n)
        s.underlying_set.is_finite
    }
}

/// Removing a newly inserted absent element recovers the original finite set.
theorem finite_set_insert_remove_of_not_contains[T](s: FiniteSet[T], item: T) {
    not s.contains(item) implies s.insert(item).remove(item) = s
} by {
    if not s.contains(item) {
        not s.underlying_set.contains(item)
        fs_insert(s, item).underlying_set = s.underlying_set.insert(item)
        fs_remove(fs_insert(s, item), item).underlying_set = fs_insert(s, item).underlying_set.remove(item)
        fs_remove(fs_insert(s, item), item).underlying_set = s.underlying_set.insert(item).remove(item)
        insert_then_remove(s.underlying_set, item)
        s.underlying_set.insert(item).remove(item) = s.underlying_set
        fs_remove(fs_insert(s, item), item).underlying_set = s.underlying_set
        finite_set_ext(fs_remove(fs_insert(s, item), item), s)
        fs_remove(fs_insert(s, item), item) = s
        s.insert(item) = fs_insert(s, item)
        s.insert(item).remove(item) = fs_remove(s.insert(item), item)
        s.insert(item).remove(item) = fs_remove(fs_insert(s, item), item)
        s.insert(item).remove(item) = s
    }
}

/// Inserting a new element increases exact cardinality by one.
theorem finite_set_insert_cardinality_is_suc_of_not_contains[T](s: FiniteSet[T], item: T, n: Nat) {
    s.cardinality_is(n) and not s.contains(item) implies fs_insert(s, item).cardinality_is(n + Nat.1)
} by {
    if s.cardinality_is(n) and not s.contains(item) {
        s.underlying_set.cardinality_is(n)
        not s.underlying_set.contains(item)
        insert_cardinality_is_suc_of_not_contains(s.underlying_set, item, n)
        s.underlying_set.insert(item).cardinality_is(n + Nat.1)
        fs_insert(s, item).underlying_set = s.underlying_set.insert(item)
        fs_insert(s, item).underlying_set.cardinality_is(n + Nat.1)
        fs_insert(s, item).cardinality_is(n + Nat.1)
    }
}

/// Removing an element and adding it back recovers the exact cardinality.
theorem finite_set_remove_insert_cardinality_is_suc_of_contains[T](s: FiniteSet[T], item: T, n: Nat) {
    fs_remove(s, item).cardinality_is(n) and s.contains(item) implies s.cardinality_is(n + Nat.1)
} by {
    if fs_remove(s, item).cardinality_is(n) and s.contains(item) {
        fs_remove(s, item).underlying_set = s.underlying_set.remove(item)
        s.underlying_set.remove(item).cardinality_is(n)
        s.underlying_set.contains(item)
        remove_insert_cardinality_is_suc_of_contains(s.underlying_set, item, n)
        s.underlying_set.cardinality_is(n + Nat.1)
        s.cardinality_is(n + Nat.1)
    }
}

/// Removing an absent element preserves exact cardinality.
theorem finite_set_remove_cardinality_is_of_not_contains[T](s: FiniteSet[T], item: T, n: Nat) {
    s.cardinality_is(n) and not s.contains(item) implies fs_remove(s, item).cardinality_is(n)
} by {
    if s.cardinality_is(n) and not s.contains(item) {
        s.underlying_set.cardinality_is(n)
        not s.underlying_set.contains(item)
        remove_cardinality_is_of_not_contains(s.underlying_set, item, n)
        s.underlying_set.remove(item).cardinality_is(n)
        fs_remove(s, item).underlying_set = s.underlying_set.remove(item)
        fs_remove(s, item).underlying_set.cardinality_is(n)
        fs_remove(s, item).cardinality_is(n)
    }
}

/// Removing a present element decreases exact cardinality by one.
theorem finite_set_remove_cardinality_is_pred_of_contains[T](s: FiniteSet[T], item: T, n: Nat) {
    s.cardinality_is(n + Nat.1) and s.contains(item) implies fs_remove(s, item).cardinality_is(n)
} by {
    if s.cardinality_is(n + Nat.1) and s.contains(item) {
        s.underlying_set.cardinality_is(n + Nat.1)
        s.underlying_set.contains(item)
        remove_cardinality_is_pred_of_contains(s.underlying_set, item, n)
        s.underlying_set.remove(item).cardinality_is(n)
        fs_remove(s, item).underlying_set = s.underlying_set.remove(item)
        fs_remove(s, item).underlying_set.cardinality_is(n)
        fs_remove(s, item).cardinality_is(n)
    }
}

/// Intersecting a finite set with one of its supersets preserves exact cardinality.
theorem finite_set_intersection_cardinality_is_of_subset_left[T](s: FiniteSet[T], t: FiniteSet[T], n: Nat) {
    s.subset_eq(t) and s.cardinality_is(n) implies fs_intersection(s, t).cardinality_is(n)
} by {
    if s.subset_eq(t) and s.cardinality_is(n) {
        s.underlying_set.subset(t.underlying_set)
        s.underlying_set.cardinality_is(n)
        intersection_cardinality_is_of_subset_left(s.underlying_set, t.underlying_set, n)
        s.underlying_set.intersection(t.underlying_set).cardinality_is(n)
        fs_intersection(s, t).underlying_set = s.underlying_set.intersection(t.underlying_set)
        fs_intersection(s, t).underlying_set.cardinality_is(n)
        fs_intersection(s, t).cardinality_is(n)
    }
}

/// Intersecting a finite set with one of its subsets has the subset's exact cardinality.
theorem finite_set_intersection_cardinality_is_of_subset_right[T](s: FiniteSet[T], t: FiniteSet[T], n: Nat) {
    t.subset_eq(s) and t.cardinality_is(n) implies fs_intersection(s, t).cardinality_is(n)
} by {
    if t.subset_eq(s) and t.cardinality_is(n) {
        t.underlying_set.subset(s.underlying_set)
        t.underlying_set.cardinality_is(n)
        intersection_cardinality_is_of_subset_right(s.underlying_set, t.underlying_set, n)
        s.underlying_set.intersection(t.underlying_set).cardinality_is(n)
        fs_intersection(s, t).underlying_set = s.underlying_set.intersection(t.underlying_set)
        fs_intersection(s, t).underlying_set.cardinality_is(n)
        fs_intersection(s, t).cardinality_is(n)
    }
}

/// Difference by a disjoint finite set preserves exact cardinality.
theorem finite_set_difference_cardinality_is_of_disjoint[T](s: FiniteSet[T], t: FiniteSet[T], n: Nat) {
    s.cardinality_is(n) and s.is_disjoint(t) implies fs_difference(s, t).cardinality_is(n)
} by {
    if s.cardinality_is(n) and s.is_disjoint(t) {
        s.underlying_set.cardinality_is(n)
        s.underlying_set.is_disjoint(t.underlying_set)
        difference_cardinality_is_of_disjoint(s.underlying_set, t.underlying_set, n)
        s.underlying_set.difference(t.underlying_set).cardinality_is(n)
        fs_difference(s, t).underlying_set = s.underlying_set.difference(t.underlying_set)
        fs_difference(s, t).underlying_set.cardinality_is(n)
        fs_difference(s, t).cardinality_is(n)
    }
}

/// Difference by a finite superset has cardinality zero.
theorem finite_set_difference_cardinality_is_zero_of_subset[T](s: FiniteSet[T], t: FiniteSet[T]) {
    s.subset_eq(t) implies fs_difference(s, t).cardinality_is(Nat.0)
} by {
    if s.subset_eq(t) {
        s.underlying_set.subset(t.underlying_set)
        difference_cardinality_is_zero_of_subset(s.underlying_set, t.underlying_set)
        s.underlying_set.difference(t.underlying_set).cardinality_is(Nat.0)
        fs_difference(s, t).underlying_set = s.underlying_set.difference(t.underlying_set)
        fs_difference(s, t).underlying_set.cardinality_is(Nat.0)
        fs_difference(s, t).cardinality_is(Nat.0)
    }
}

/// Disjoint finite sets have union cardinality equal to the sum of their cardinalities.
theorem finite_set_disjoint_union_cardinality_is[T](a: FiniteSet[T], b: FiniteSet[T], n1: Nat, n2: Nat) {
    a.cardinality_is(n1) and b.cardinality_is(n2) and a.is_disjoint(b)
    implies fs_union(a, b).cardinality_is(n1 + n2)
} by {
    if a.cardinality_is(n1) and b.cardinality_is(n2) and a.is_disjoint(b) {
        a.underlying_set.cardinality_is(n1)
        b.underlying_set.cardinality_is(n2)
        a.underlying_set.is_disjoint(b.underlying_set)
        disjoint_union_is_length(a.underlying_set, b.underlying_set, n1, n2)
        a.underlying_set.union(b.underlying_set).cardinality_is(n1 + n2)
        fs_union(a, b).underlying_set = a.underlying_set.union(b.underlying_set)
        fs_union(a, b).underlying_set.cardinality_is(n1 + n2)
        fs_union(a, b).cardinality_is(n1 + n2)
    }
}

/// A union decomposed into a set and the outside difference has additive cardinality.
theorem finite_set_union_cardinality_with_difference[T](s: FiniteSet[T], t: FiniteSet[T], n_s: Nat, n_diff: Nat) {
    s.cardinality_is(n_s) and fs_difference(t, s).cardinality_is(n_diff)
    implies fs_union(s, t).cardinality_is(n_s + n_diff)
} by {
    if s.cardinality_is(n_s) and fs_difference(t, s).cardinality_is(n_diff) {
        s.underlying_set.cardinality_is(n_s)
        fs_difference(t, s).underlying_set = t.underlying_set.difference(s.underlying_set)
        t.underlying_set.difference(s.underlying_set).cardinality_is(n_diff)
        union_cardinality_with_difference(s.underlying_set, t.underlying_set, n_s, n_diff)
        s.underlying_set.union(t.underlying_set).cardinality_is(n_s + n_diff)
        fs_union(s, t).underlying_set = s.underlying_set.union(t.underlying_set)
        fs_union(s, t).underlying_set.cardinality_is(n_s + n_diff)
        fs_union(s, t).cardinality_is(n_s + n_diff)
    }
}

/// The intersection of finite sets is disjoint from the left difference.
theorem finite_set_intersection_is_disjoint_difference[T](s: FiniteSet[T], t: FiniteSet[T]) {
    fs_intersection(s, t).is_disjoint(fs_difference(s, t))
} by {
    fs_intersection(s, t).underlying_set = s.underlying_set.intersection(t.underlying_set)
    fs_difference(s, t).underlying_set = s.underlying_set.difference(t.underlying_set)
    intersection_is_disjoint_difference(s.underlying_set, t.underlying_set)
    s.underlying_set.intersection(t.underlying_set).is_disjoint(
        s.underlying_set.difference(t.underlying_set))
    fs_intersection(s, t).underlying_set.is_disjoint(fs_difference(s, t).underlying_set)
    fs_intersection(s, t).is_disjoint(fs_difference(s, t))
}

/// The union of the finite-set intersection with the left difference is the left finite set.
theorem finite_set_intersection_union_difference_is_self[T](s: FiniteSet[T], t: FiniteSet[T]) {
    fs_union(fs_intersection(s, t), fs_difference(s, t)) = s
} by {
    fs_intersection(s, t).underlying_set = s.underlying_set.intersection(t.underlying_set)
    fs_difference(s, t).underlying_set = s.underlying_set.difference(t.underlying_set)
    fs_union(fs_intersection(s, t), fs_difference(s, t)).underlying_set =
        fs_intersection(s, t).underlying_set.union(fs_difference(s, t).underlying_set)
    intersection_union_difference_is_self(s.underlying_set, t.underlying_set)
    s.underlying_set.intersection(t.underlying_set).union(
        s.underlying_set.difference(t.underlying_set)) = s.underlying_set
    fs_union(fs_intersection(s, t), fs_difference(s, t)).underlying_set = s.underlying_set
    finite_set_ext(fs_union(fs_intersection(s, t), fs_difference(s, t)), s)
    fs_union(fs_intersection(s, t), fs_difference(s, t)) = s
}

/// Exact cardinalities for the finite-set intersection and left difference add to the left cardinality.
theorem finite_set_intersection_difference_cardinality_adds[T](s: FiniteSet[T], t: FiniteSet[T],
    n_inter: Nat, n_diff: Nat) {
    fs_intersection(s, t).cardinality_is(n_inter) and fs_difference(s, t).cardinality_is(n_diff)
    implies s.cardinality_is(n_inter + n_diff)
} by {
    if fs_intersection(s, t).cardinality_is(n_inter) and fs_difference(s, t).cardinality_is(n_diff) {
        fs_intersection(s, t).underlying_set = s.underlying_set.intersection(t.underlying_set)
        fs_difference(s, t).underlying_set = s.underlying_set.difference(t.underlying_set)
        s.underlying_set.intersection(t.underlying_set).cardinality_is(n_inter)
        s.underlying_set.difference(t.underlying_set).cardinality_is(n_diff)
        intersection_difference_cardinality_adds(s.underlying_set, t.underlying_set, n_inter, n_diff)
        s.underlying_set.cardinality_is(n_inter + n_diff)
        s.cardinality_is(n_inter + n_diff)
    }
}

/// The finite-set left difference has cardinality equal to the left cardinality minus the intersection cardinality.
theorem finite_set_difference_cardinality_is_sub_intersection[T](s: FiniteSet[T], t: FiniteSet[T],
    n_s: Nat, n_inter: Nat) {
    s.cardinality_is(n_s) and fs_intersection(s, t).cardinality_is(n_inter)
    implies fs_difference(s, t).cardinality_is(n_s - n_inter)
} by {
    if s.cardinality_is(n_s) and fs_intersection(s, t).cardinality_is(n_inter) {
        s.underlying_set.cardinality_is(n_s)
        fs_intersection(s, t).underlying_set = s.underlying_set.intersection(t.underlying_set)
        s.underlying_set.intersection(t.underlying_set).cardinality_is(n_inter)
        difference_cardinality_is_sub_intersection(s.underlying_set, t.underlying_set, n_s, n_inter)
        s.underlying_set.difference(t.underlying_set).cardinality_is(n_s - n_inter)
        fs_difference(s, t).underlying_set = s.underlying_set.difference(t.underlying_set)
        fs_difference(s, t).underlying_set.cardinality_is(n_s - n_inter)
        fs_difference(s, t).cardinality_is(n_s - n_inter)
    }
}

/// The finite-set intersection has cardinality equal to the left cardinality minus the left-difference cardinality.
theorem finite_set_intersection_cardinality_is_sub_difference[T](s: FiniteSet[T], t: FiniteSet[T],
    n_s: Nat, n_diff: Nat) {
    s.cardinality_is(n_s) and fs_difference(s, t).cardinality_is(n_diff)
    implies fs_intersection(s, t).cardinality_is(n_s - n_diff)
} by {
    if s.cardinality_is(n_s) and fs_difference(s, t).cardinality_is(n_diff) {
        s.underlying_set.cardinality_is(n_s)
        fs_difference(s, t).underlying_set = s.underlying_set.difference(t.underlying_set)
        s.underlying_set.difference(t.underlying_set).cardinality_is(n_diff)
        intersection_cardinality_is_sub_difference(s.underlying_set, t.underlying_set, n_s, n_diff)
        s.underlying_set.intersection(t.underlying_set).cardinality_is(n_s - n_diff)
        fs_intersection(s, t).underlying_set = s.underlying_set.intersection(t.underlying_set)
        fs_intersection(s, t).underlying_set.cardinality_is(n_s - n_diff)
        fs_intersection(s, t).cardinality_is(n_s - n_diff)
    }
}

/// Inclusion-exclusion for the union of two finite sets.
theorem finite_set_inclusion_exclusion[T](s: FiniteSet[T], t: FiniteSet[T],
    n_s: Nat, n_t: Nat, n_inter: Nat) {
    s.cardinality_is(n_s) and t.cardinality_is(n_t) and fs_intersection(s, t).cardinality_is(n_inter)
    implies fs_union(s, t).cardinality_is(n_s + n_t - n_inter)
} by {
    if s.cardinality_is(n_s) and t.cardinality_is(n_t) and fs_intersection(s, t).cardinality_is(n_inter) {
        s.underlying_set.cardinality_is(n_s)
        t.underlying_set.cardinality_is(n_t)
        fs_intersection(s, t).underlying_set = s.underlying_set.intersection(t.underlying_set)
        s.underlying_set.intersection(t.underlying_set).cardinality_is(n_inter)
        inclusion_exclusion(s.underlying_set, t.underlying_set, n_s, n_t, n_inter)
        s.underlying_set.union(t.underlying_set).cardinality_is(n_s + n_t - n_inter)
        fs_union(s, t).underlying_set = s.underlying_set.union(t.underlying_set)
        fs_union(s, t).underlying_set.cardinality_is(n_s + n_t - n_inter)
        fs_union(s, t).cardinality_is(n_s + n_t - n_inter)
    }
}

/// The union of two pairwise intersections has the inclusion-exclusion cardinality.
theorem finite_set_pair_intersections_union_cardinality[T](
    a: FiniteSet[T], b: FiniteSet[T], c: FiniteSet[T],
    n_ac: Nat, n_bc: Nat, n_abc: Nat
) {
    fs_intersection(a, c).cardinality_is(n_ac) and
    fs_intersection(b, c).cardinality_is(n_bc) and
    fs_intersection(fs_intersection(a, b), c).cardinality_is(n_abc)
    implies fs_union(fs_intersection(a, c), fs_intersection(b, c)).cardinality_is(
        n_ac + n_bc - n_abc)
} by {
    if fs_intersection(a, c).cardinality_is(n_ac) and
        fs_intersection(b, c).cardinality_is(n_bc) and
        fs_intersection(fs_intersection(a, b), c).cardinality_is(n_abc) {
        let ac_inter = fs_intersection(a, c)
        let bc_inter = fs_intersection(b, c)
        let abc_inter = fs_intersection(fs_intersection(a, b), c)
        ac_inter.cardinality_is(n_ac)
        bc_inter.cardinality_is(n_bc)
        abc_inter.cardinality_is(n_abc)
        finite_set_pair_intersections(a, b, c)
        fs_intersection(ac_inter, bc_inter) = abc_inter
        fs_intersection(ac_inter, bc_inter).cardinality_is(n_abc)
        finite_set_inclusion_exclusion(ac_inter, bc_inter, n_ac, n_bc, n_abc)
    }
}

/// Inclusion-exclusion for the union of three finite sets.
theorem finite_set_inclusion_exclusion_three[T](
    a: FiniteSet[T], b: FiniteSet[T], c: FiniteSet[T],
    n_a: Nat, n_b: Nat, n_c: Nat,
    n_ab: Nat, n_ac: Nat, n_bc: Nat, n_abc: Nat
) {
    a.cardinality_is(n_a) and b.cardinality_is(n_b) and c.cardinality_is(n_c) and
    fs_intersection(a, b).cardinality_is(n_ab) and
    fs_intersection(a, c).cardinality_is(n_ac) and
    fs_intersection(b, c).cardinality_is(n_bc) and
    fs_intersection(fs_intersection(a, b), c).cardinality_is(n_abc)
    implies fs_union(fs_union(a, b), c).cardinality_is(
        (n_a + n_b - n_ab) + n_c - (n_ac + n_bc - n_abc))
} by {
    if a.cardinality_is(n_a) and b.cardinality_is(n_b) and c.cardinality_is(n_c) and
        fs_intersection(a, b).cardinality_is(n_ab) and
        fs_intersection(a, c).cardinality_is(n_ac) and
        fs_intersection(b, c).cardinality_is(n_bc) and
        fs_intersection(fs_intersection(a, b), c).cardinality_is(n_abc) {
        a.cardinality_is(n_a)
        b.cardinality_is(n_b)
        c.cardinality_is(n_c)
        fs_intersection(a, b).cardinality_is(n_ab)
        fs_intersection(a, c).cardinality_is(n_ac)
        fs_intersection(b, c).cardinality_is(n_bc)
        fs_intersection(fs_intersection(a, b), c).cardinality_is(n_abc)

        finite_set_inclusion_exclusion(a, b, n_a, n_b, n_ab)
        fs_union(a, b).cardinality_is(n_a + n_b - n_ab)

        finite_set_pair_intersections_union_cardinality(a, b, c, n_ac, n_bc, n_abc)
        fs_union(fs_intersection(a, c), fs_intersection(b, c)).cardinality_is(
            n_ac + n_bc - n_abc)

        finite_set_intersection_union_distrib(a, b, c)
        fs_intersection(fs_union(a, b), c) =
            fs_union(fs_intersection(a, c), fs_intersection(b, c))
        fs_intersection(fs_union(a, b), c).cardinality_is(n_ac + n_bc - n_abc)
        finite_set_inclusion_exclusion(
            fs_union(a, b), c, n_a + n_b - n_ab, n_c, n_ac + n_bc - n_abc)
        fs_union(fs_union(a, b), c).cardinality_is(
            (n_a + n_b - n_ab) + n_c - (n_ac + n_bc - n_abc))
    }
}

lemma finite_set_bonferroni_lower_three_arithmetic(
    n_a: Nat, n_b: Nat, n_c: Nat,
    n_ab: Nat, n_ac: Nat, n_bc: Nat, n_abc: Nat
) {
    n_a + n_b + n_c - (n_ab + n_ac + n_bc) <=
        (n_a + n_b - n_ab) + n_c - (n_ac + n_bc - n_abc)
} by {
    let singles_ab = n_a + n_b
    let pair_ac_bc = n_ac + n_bc
    let singles = singles_ab + n_c
    let pairs = n_ab + pair_ac_bc
    let lower = singles - pairs
    let upper = (singles_ab - n_ab) + n_c - (pair_ac_bc - n_abc)
    let ab_remainder = singles_ab - n_ab
    let pair_remainder = pair_ac_bc - n_abc
    let ab_excess = n_ab - singles_ab
    if singles < pairs {
        sub_lt(singles, pairs)
        singles - pairs = Nat.0
        lower = Nat.0
        lower <= upper
    } else {
        pairs <= singles
        add_sub(singles, pairs)
        singles - pairs + pairs = singles
        lower + pairs = singles
        lower + n_ab + pair_ac_bc = singles_ab + n_c
        if singles_ab < n_ab {
            sub_lt(singles_ab, n_ab)
            singles_ab - n_ab = Nat.0
            ab_remainder = Nat.0
            add_sub(n_ab, singles_ab)
            ab_excess + singles_ab = n_ab
            if pair_ac_bc < n_abc {
                sub_lt(pair_ac_bc, n_abc)
                pair_ac_bc - n_abc = Nat.0
                pair_remainder = Nat.0
                lower + (ab_excess + singles_ab) + pair_ac_bc = singles_ab + n_c
                add_assoc(lower, ab_excess, singles_ab)
                add_comm(lower + ab_excess, singles_ab)
                add_assoc(singles_ab, lower + ab_excess, pair_ac_bc)
                singles_ab + (lower + ab_excess + pair_ac_bc) = singles_ab + n_c
                add_cancels_left(singles_ab, lower + ab_excess + pair_ac_bc, n_c)
                lower + ab_excess + pair_ac_bc = n_c
                lower <= n_c
                upper = n_c
                lower <= upper
            } else {
                n_abc <= pair_ac_bc
                add_sub(pair_ac_bc, n_abc)
                pair_ac_bc - n_abc + n_abc = pair_ac_bc
                pair_remainder + n_abc = pair_ac_bc
                lower + (ab_excess + singles_ab) + pair_ac_bc = singles_ab + n_c
                add_assoc(lower, ab_excess, singles_ab)
                add_comm(lower + ab_excess, singles_ab)
                add_assoc(singles_ab, lower + ab_excess, pair_ac_bc)
                singles_ab + (lower + ab_excess + pair_ac_bc) = singles_ab + n_c
                add_cancels_left(singles_ab, lower + ab_excess + pair_ac_bc, n_c)
                lower + ab_excess + pair_ac_bc = n_c
                lower + ab_excess + pair_remainder + n_abc = n_c
                lower + ab_excess + n_abc + pair_remainder = n_c
                add_imp_sub(lower + ab_excess + n_abc, pair_remainder, n_c)
                n_c - pair_remainder = lower + ab_excess + n_abc
                upper = lower + ab_excess + n_abc
                lower <= upper
            }
        } else {
            n_ab <= singles_ab
            add_sub(singles_ab, n_ab)
            singles_ab - n_ab + n_ab = singles_ab
            ab_remainder + n_ab = singles_ab
            if pair_ac_bc < n_abc {
                sub_lt(pair_ac_bc, n_abc)
                pair_ac_bc - n_abc = Nat.0
                pair_remainder = Nat.0
                n_ab + (lower + pair_ac_bc) = n_ab + (ab_remainder + n_c)
                add_cancels_left(n_ab, lower + pair_ac_bc, ab_remainder + n_c)
                lower + pair_ac_bc = ab_remainder + n_c
                lower <= ab_remainder + n_c
                upper = ab_remainder + n_c
                lower <= upper
            } else {
                n_abc <= pair_ac_bc
                add_sub(pair_ac_bc, n_abc)
                pair_ac_bc - n_abc + n_abc = pair_ac_bc
                pair_remainder + n_abc = pair_ac_bc
                n_ab + (lower + pair_ac_bc) = n_ab + (ab_remainder + n_c)
                add_cancels_left(n_ab, lower + pair_ac_bc, ab_remainder + n_c)
                lower + pair_ac_bc = ab_remainder + n_c
                lower + pair_remainder + n_abc = ab_remainder + n_c
                lower + n_abc + pair_remainder = ab_remainder + n_c
                add_imp_sub(lower + n_abc, pair_remainder, ab_remainder + n_c)
                ab_remainder + n_c - pair_remainder = lower + n_abc
                upper = lower + n_abc
                lower <= upper
            }
        }
    }
    lower <= upper
}

/// The sum of singleton cardinalities minus pairwise-intersection cardinalities
/// is a lower bound for the cardinality of a union of three finite sets.
theorem finite_set_bonferroni_lower_three[T](
    a: FiniteSet[T], b: FiniteSet[T], c: FiniteSet[T],
    n_a: Nat, n_b: Nat, n_c: Nat,
    n_ab: Nat, n_ac: Nat, n_bc: Nat, n_union: Nat
) {
    a.cardinality_is(n_a) and b.cardinality_is(n_b) and c.cardinality_is(n_c) and
    fs_intersection(a, b).cardinality_is(n_ab) and
    fs_intersection(a, c).cardinality_is(n_ac) and
    fs_intersection(b, c).cardinality_is(n_bc) and
    fs_union(fs_union(a, b), c).cardinality_is(n_union)
    implies n_a + n_b + n_c - (n_ab + n_ac + n_bc) <= n_union
} by {
    if a.cardinality_is(n_a) and b.cardinality_is(n_b) and c.cardinality_is(n_c) and
        fs_intersection(a, b).cardinality_is(n_ab) and
        fs_intersection(a, c).cardinality_is(n_ac) and
        fs_intersection(b, c).cardinality_is(n_bc) and
        fs_union(fs_union(a, b), c).cardinality_is(n_union) {
        a.cardinality_is(n_a)
        b.cardinality_is(n_b)
        c.cardinality_is(n_c)
        fs_intersection(a, b).cardinality_is(n_ab)
        fs_intersection(a, c).cardinality_is(n_ac)
        fs_intersection(b, c).cardinality_is(n_bc)
        fs_union(fs_union(a, b), c).cardinality_is(n_union)
        let abc_inter = fs_intersection(fs_intersection(a, b), c)
        finite_set_cardinality_exists(abc_inter)
        let n_abc: Nat satisfy {
            abc_inter.cardinality_is(n_abc)
        }
        finite_set_inclusion_exclusion_three(
            a, b, c, n_a, n_b, n_c, n_ab, n_ac, n_bc, n_abc)
        fs_union(fs_union(a, b), c).cardinality_is(
            (n_a + n_b - n_ab) + n_c - (n_ac + n_bc - n_abc))
        finite_set_cardinality_is_well_defined(
            fs_union(fs_union(a, b), c),
            (n_a + n_b - n_ab) + n_c - (n_ac + n_bc - n_abc), n_union)
        (n_a + n_b - n_ab) + n_c - (n_ac + n_bc - n_abc) = n_union
        finite_set_bonferroni_lower_three_arithmetic(
            n_a, n_b, n_c, n_ab, n_ac, n_bc, n_abc)
        n_a + n_b + n_c - (n_ab + n_ac + n_bc) <= n_union
    }
}
