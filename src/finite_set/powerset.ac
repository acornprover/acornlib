from finite_set.base import FiniteSet, finite_set_cardinality_is_well_defined,
    finite_set_from_unique_list_cardinality_is_length, finite_set_has_unique_list,
    fs_from_list
from list import List
from finite_set.list_subsets import finite_powerset_from_list,
    finite_powerset_from_list_cardinality, finite_powerset_from_list_exact
from nat import Nat

/// The powerset of a finite set, represented through a duplicate-free list of its elements.
let finite_powerset[T](s: FiniteSet[T]) -> result: FiniteSet[FiniteSet[T]] satisfy {
    exists(items: List[T]) {
        fs_from_list(items) = s and items.is_unique and
        result = finite_powerset_from_list[T](items)
    }
} by {
    finite_set_has_unique_list[T](s)
    let items: List[T] satisfy {
        fs_from_list[T](items) = s and items.is_unique
    }
    let powerset = finite_powerset_from_list[T](items)
}

/// The finite powerset contains exactly the finite subsets of its base set.
theorem finite_powerset_exact[T](s: FiniteSet[T]) {
    forall(t: FiniteSet[T]) {
        finite_powerset[T](s).contains(t) = t.subset_eq(s)
    }
} by {
    let items: List[T] satisfy {
        fs_from_list[T](items) = s and items.is_unique and
        finite_powerset[T](s) = finite_powerset_from_list[T](items)
    }
    finite_powerset_from_list_exact[T](items)
    forall(t: FiniteSet[T]) {
        t.subset_eq(FiniteSet.from_list[T](items)) = t.subset_eq(s)
        finite_powerset[T](s).contains(t) = t.subset_eq(s)
    }
}

/// A finite set of cardinality `n` has a powerset of cardinality `2 ^ n`.
theorem finite_powerset_cardinality[T](s: FiniteSet[T], n: Nat) {
    s.cardinality_is(n) implies finite_powerset[T](s).cardinality_is(Nat.2.pow(n))
} by {
    let items: List[T] satisfy {
        fs_from_list[T](items) = s and items.is_unique and
        finite_powerset[T](s) = finite_powerset_from_list[T](items)
    }
    finite_set_from_unique_list_cardinality_is_length[T](items)
    finite_set_cardinality_is_well_defined[T](fs_from_list[T](items), items.length, n)
    items.length = n
    finite_powerset_from_list_cardinality[T](items)
    finite_powerset_from_list[T](items).cardinality_is(Nat.2.pow(items.length))
}
