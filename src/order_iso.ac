/// Order isomorphisms between partially ordered types.

from data.basic.functions import compose, identity_fn, function_extensionality,
    is_injective_fn, is_surjective_fn, is_bijection_fn
from order import PartialOrder, LinearOrder, lt_trans, min_eq_left_of_lte, min_eq_right_of_gte,
    max_eq_left_of_gte, max_eq_right_of_lte
from order import closed_interval, open_interval, left_open_interval, right_open_interval,
    is_lower_bound, is_upper_bound, is_bounded_below, is_bounded_above, is_bounded_by_interval,
    is_bounded, lower_bound_step, upper_bound_step, bounds_imp_bounded_by_interval,
    bounded_by_interval_imp_lower_bound, bounded_by_interval_imp_upper_bound,
    lower_bound_imp_bounded_below, upper_bound_imp_bounded_above,
    bounded_by_interval_imp_bounded
from order import is_antitone, is_strict_antitone, antitone_step, antitone_apply, is_order_embedding, order_embedding_apply_le,
    order_embedding_apply_lt, order_embedding_apply_ge, order_embedding_apply_gt,
    order_embedding_le_iff_le,
    order_embedding_reflects_lte, order_embedding_reflects_lt, order_embedding_reflects_ge,
    order_embedding_reflects_gt, identity_is_order_embedding, order_embedding_compose,
    antitone_closed_interval_image, strict_antitone_open_interval_image,
    antitone_left_open_interval_image, antitone_right_open_interval_image,
    order_embedding_closed_interval_image, order_embedding_closed_interval_preimage,
    order_embedding_closed_interval_iff, order_embedding_open_interval_image,
    order_embedding_open_interval_preimage, order_embedding_open_interval_iff,
    order_embedding_left_open_interval_image, order_embedding_left_open_interval_preimage,
    order_embedding_left_open_interval_iff, order_embedding_right_open_interval_image,
    order_embedding_right_open_interval_preimage, order_embedding_right_open_interval_iff,
    order_embedding_min_image, order_embedding_max_image
from lattice import MeetSemilattice, JoinSemilattice, meet_lte_left, meet_lte_right,
    lte_meet_of_bounds, lte_join_left, lte_join_right, join_lte_of_bounds
from algebra.field.galois_connection import is_galois_connection, is_galois_insertion, is_galois_coinsertion

/// True if two maps are inverse order-preserving maps.
define is_order_iso_pair[A: PartialOrder, B: PartialOrder](f: A -> B, g: B -> A) -> Bool {
    forall(a: A) {
        g(f(a)) = a
    } and forall(b: B) {
        f(g(b)) = b
    } and is_order_embedding(f) and is_order_embedding(g)
}

/// An order isomorphism pair has a left inverse.
theorem order_iso_pair_left_inverse[A: PartialOrder, B: PartialOrder](f: A -> B, g: B -> A, a: A) {
    is_order_iso_pair(f, g) implies g(f(a)) = a
} by {
    if is_order_iso_pair(f, g) {
        is_order_iso_pair(f, g) = (forall(x: A) {
            g(f(x)) = x
        } and forall(y: B) {
            f(g(y)) = y
        } and is_order_embedding(f) and is_order_embedding(g))
        g(f(a)) = a
    }
}

/// An order isomorphism pair has a right inverse.
theorem order_iso_pair_right_inverse[A: PartialOrder, B: PartialOrder](f: A -> B, g: B -> A, b: B) {
    is_order_iso_pair(f, g) implies f(g(b)) = b
} by {
    if is_order_iso_pair(f, g) {
        is_order_iso_pair(f, g) = (forall(x: A) {
            g(f(x)) = x
        } and forall(y: B) {
            f(g(y)) = y
        } and is_order_embedding(f) and is_order_embedding(g))
        f(g(b)) = b
    }
}

/// The forward map of an order isomorphism pair is an order embedding.
theorem order_iso_pair_map_is_order_embedding[A: PartialOrder, B: PartialOrder](f: A -> B, g: B -> A) {
    is_order_iso_pair(f, g) implies is_order_embedding(f)
} by {
    if is_order_iso_pair(f, g) {
        is_order_iso_pair(f, g) = (forall(x: A) {
            g(f(x)) = x
        } and forall(y: B) {
            f(g(y)) = y
        } and is_order_embedding(f) and is_order_embedding(g))
        is_order_embedding(f)
    }
}

/// The inverse map of an order isomorphism pair is an order embedding.
theorem order_iso_pair_inv_is_order_embedding[A: PartialOrder, B: PartialOrder](f: A -> B, g: B -> A) {
    is_order_iso_pair(f, g) implies is_order_embedding(g)
} by {
    if is_order_iso_pair(f, g) {
        is_order_iso_pair(f, g) = (forall(x: A) {
            g(f(x)) = x
        } and forall(y: B) {
            f(g(y)) = y
        } and is_order_embedding(f) and is_order_embedding(g))
        is_order_embedding(g)
    }
}

/// Reversing an order isomorphism pair gives an order isomorphism pair.
theorem order_iso_pair_swap[A: PartialOrder, B: PartialOrder](f: A -> B, g: B -> A) {
    is_order_iso_pair(f, g) implies is_order_iso_pair(g, f)
} by {
    if is_order_iso_pair(f, g) {
        is_order_iso_pair(f, g) = (forall(x: A) {
            g(f(x)) = x
        } and forall(y: B) {
            f(g(y)) = y
        } and is_order_embedding(f) and is_order_embedding(g))
        forall(b: B) {
            f(g(b)) = b
        }
        forall(a: A) {
            g(f(a)) = a
        }
        is_order_embedding(g)
        is_order_embedding(f)
        is_order_iso_pair(g, f) = (forall(x: B) {
            f(g(x)) = x
        } and forall(y: A) {
            g(f(y)) = y
        } and is_order_embedding(g) and is_order_embedding(f))
        is_order_iso_pair(g, f)
    }
}

/// Composing order isomorphism pairs gives an order isomorphism pair.
theorem order_iso_pair_compose[A: PartialOrder, B: PartialOrder, C: PartialOrder](
    f: B -> C,
    g: C -> B,
    h: A -> B,
    k: B -> A
) {
    is_order_iso_pair(f, g) and is_order_iso_pair(h, k) implies
    is_order_iso_pair(compose(f, h), compose(k, g))
} by {
    if is_order_iso_pair(f, g) and is_order_iso_pair(h, k) {
        let fg = compose(f, h)
        let kg = compose(k, g)
        is_order_iso_pair(f, g) = (forall(b: B) {
            g(f(b)) = b
        } and forall(c: C) {
            f(g(c)) = c
        } and is_order_embedding(f) and is_order_embedding(g))
        is_order_iso_pair(h, k) = (forall(a: A) {
            k(h(a)) = a
        } and forall(b: B) {
            h(k(b)) = b
        } and is_order_embedding(h) and is_order_embedding(k))
        order_embedding_compose(h, f)
        is_order_embedding(fg)
        order_embedding_compose(g, k)
        is_order_embedding(kg)
        forall(a: A) {
            kg(fg(a)) = compose(k, g, fg(a))
            compose(k, g, fg(a)) = k(g(fg(a)))
            fg(a) = compose(f, h)(a)
            compose(f, h, a) = f(h(a))
            g(fg(a)) = g(f(h(a)))
            g(f(h(a))) = h(a)
            k(g(fg(a))) = k(h(a))
            k(h(a)) = a
            kg(fg(a)) = a
        }
        forall(c: C) {
            fg(kg(c)) = compose(f, h, kg(c))
            compose(f, h, kg(c)) = f(h(kg(c)))
            kg(c) = compose(k, g)(c)
            compose(k, g, c) = k(g(c))
            h(kg(c)) = h(k(g(c)))
            h(k(g(c))) = g(c)
            f(h(kg(c))) = f(g(c))
            f(g(c)) = c
            fg(kg(c)) = c
        }
        is_order_iso_pair(fg, kg) = (forall(a: A) {
            kg(fg(a)) = a
        } and forall(c: C) {
            fg(kg(c)) = c
        } and is_order_embedding(fg) and is_order_embedding(kg))
        forall(x: A) {
            kg(fg(x)) = x
        }
        forall(y: C) {
            fg(kg(y)) = y
        }
        is_order_embedding(fg) and is_order_embedding(kg)
        (forall(y: C) {
            fg(kg(y)) = y
        } and is_order_embedding(fg) and is_order_embedding(kg))
        (forall(x: A) {
            kg(fg(x)) = x
        } and forall(y: C) {
            fg(kg(y)) = y
        } and is_order_embedding(fg) and is_order_embedding(kg))
        is_order_iso_pair(fg, kg)
        fg = compose(f, h)
        kg = compose(k, g)
        is_order_iso_pair(compose(f, h), compose(k, g))
    }
}

/// An order isomorphism with explicit inverse data.
structure OrderIso[A: PartialOrder, B: PartialOrder] {
    /// The order-preserving map.
    map: A -> B

    /// The inverse order-preserving map.
    inv: B -> A
} constraint {
    is_order_iso_pair(map, inv)
}

/// True if two maps are inverse order-reversing maps.
define is_order_dual_iso_pair[A: PartialOrder, B: PartialOrder](f: A -> B, g: B -> A) -> Bool {
    forall(a: A) {
        g(f(a)) = a
    } and forall(b: B) {
        f(g(b)) = b
    } and is_antitone(f) and is_antitone(g)
}

/// An order-dual isomorphism pair has a left inverse.
theorem order_dual_iso_pair_left_inverse[A: PartialOrder, B: PartialOrder](f: A -> B, g: B -> A, a: A) {
    is_order_dual_iso_pair(f, g) implies g(f(a)) = a
} by {
    if is_order_dual_iso_pair(f, g) {
        is_order_dual_iso_pair(f, g) = (forall(x: A) {
            g(f(x)) = x
        } and forall(y: B) {
            f(g(y)) = y
        } and is_antitone(f) and is_antitone(g))
        g(f(a)) = a
    }
}

/// An order-dual isomorphism pair has a right inverse.
theorem order_dual_iso_pair_right_inverse[A: PartialOrder, B: PartialOrder](f: A -> B, g: B -> A, b: B) {
    is_order_dual_iso_pair(f, g) implies f(g(b)) = b
} by {
    if is_order_dual_iso_pair(f, g) {
        is_order_dual_iso_pair(f, g) = (forall(x: A) {
            g(f(x)) = x
        } and forall(y: B) {
            f(g(y)) = y
        } and is_antitone(f) and is_antitone(g))
        f(g(b)) = b
    }
}

/// The forward map of an order-dual isomorphism pair is antitone.
theorem order_dual_iso_pair_map_is_antitone[A: PartialOrder, B: PartialOrder](f: A -> B, g: B -> A) {
    is_order_dual_iso_pair(f, g) implies is_antitone(f)
} by {
    if is_order_dual_iso_pair(f, g) {
        is_order_dual_iso_pair(f, g) = (forall(x: A) {
            g(f(x)) = x
        } and forall(y: B) {
            f(g(y)) = y
        } and is_antitone(f) and is_antitone(g))
        is_antitone(f)
    }
}

/// The inverse map of an order-dual isomorphism pair is antitone.
theorem order_dual_iso_pair_inv_is_antitone[A: PartialOrder, B: PartialOrder](f: A -> B, g: B -> A) {
    is_order_dual_iso_pair(f, g) implies is_antitone(g)
} by {
    if is_order_dual_iso_pair(f, g) {
        is_order_dual_iso_pair(f, g) = (forall(x: A) {
            g(f(x)) = x
        } and forall(y: B) {
            f(g(y)) = y
        } and is_antitone(f) and is_antitone(g))
        is_antitone(g)
    }
}

/// Reversing an order-dual isomorphism pair gives an order-dual isomorphism pair.
theorem order_dual_iso_pair_swap[A: PartialOrder, B: PartialOrder](f: A -> B, g: B -> A) {
    is_order_dual_iso_pair(f, g) implies is_order_dual_iso_pair(g, f)
} by {
    if is_order_dual_iso_pair(f, g) {
        is_order_dual_iso_pair(f, g) = (forall(x: A) {
            g(f(x)) = x
        } and forall(y: B) {
            f(g(y)) = y
        } and is_antitone(f) and is_antitone(g))
        forall(b: B) {
            f(g(b)) = b
        }
        forall(a: A) {
            g(f(a)) = a
        }
        is_antitone(g)
        is_antitone(f)
        is_order_dual_iso_pair(g, f) = (forall(x: B) {
            f(g(x)) = x
        } and forall(y: A) {
            g(f(y)) = y
        } and is_antitone(g) and is_antitone(f))
        is_order_dual_iso_pair(g, f)
    }
}

/// An order-dual isomorphism pair reverses non-strict order.
theorem order_dual_iso_pair_apply_le[A: PartialOrder, B: PartialOrder](f: A -> B, g: B -> A, x: A, y: A) {
    is_order_dual_iso_pair(f, g) and x <= y implies f(y) <= f(x)
} by {
    if is_order_dual_iso_pair(f, g) and x <= y {
        order_dual_iso_pair_map_is_antitone(f, g)
        antitone_step(f, x, y)
        f(y) <= f(x)
    }
}

/// An order-dual isomorphism pair reflects reversed non-strict order.
theorem order_dual_iso_pair_reflects_le[A: PartialOrder, B: PartialOrder](f: A -> B, g: B -> A, x: A, y: A) {
    is_order_dual_iso_pair(f, g) and f(y) <= f(x) implies x <= y
} by {
    if is_order_dual_iso_pair(f, g) and f(y) <= f(x) {
        order_dual_iso_pair_inv_is_antitone(f, g)
        antitone_step(g, f(y), f(x))
        g(f(x)) <= g(f(y))
        order_dual_iso_pair_left_inverse(f, g, x)
        order_dual_iso_pair_left_inverse(f, g, y)
        g(f(x)) = x
        g(f(y)) = y
        x <= y
    }
}

/// An order-dual isomorphism pair reverses and reflects non-strict order.
theorem order_dual_iso_pair_le_iff_ge[A: PartialOrder, B: PartialOrder](f: A -> B, g: B -> A, x: A, y: A) {
    is_order_dual_iso_pair(f, g) implies (f(x) <= f(y) = (x >= y))
} by {
    if is_order_dual_iso_pair(f, g) {
        if f(x) <= f(y) {
            order_dual_iso_pair_reflects_le(f, g, y, x)
            y <= x
            x >= y
        }
        if x >= y {
            y <= x
            order_dual_iso_pair_apply_le(f, g, y, x)
            f(x) <= f(y)
        }
        f(x) <= f(y) = (x >= y)
    }
}

/// An order-dual isomorphism pair reverses strict order.
theorem order_dual_iso_pair_apply_lt[A: PartialOrder, B: PartialOrder](f: A -> B, g: B -> A, x: A, y: A) {
    is_order_dual_iso_pair(f, g) and x < y implies f(y) < f(x)
} by {
    if is_order_dual_iso_pair(f, g) and x < y {
        x <= y
        order_dual_iso_pair_apply_le(f, g, x, y)
        f(y) <= f(x)
        if f(y) = f(x) {
            g(f(y)) = g(f(x))
            order_dual_iso_pair_left_inverse(f, g, y)
            order_dual_iso_pair_left_inverse(f, g, x)
            y = x
            x = y
            x != y
            false
        }
        f(y) != f(x)
        f(y) < f(x)
    }
}

/// An order-dual isomorphism pair reflects reversed strict order.
theorem order_dual_iso_pair_reflects_lt[A: PartialOrder, B: PartialOrder](f: A -> B, g: B -> A, x: A, y: A) {
    is_order_dual_iso_pair(f, g) and f(y) < f(x) implies x < y
} by {
    if is_order_dual_iso_pair(f, g) and f(y) < f(x) {
        f(y) <= f(x)
        order_dual_iso_pair_reflects_le(f, g, x, y)
        x <= y
        if x = y {
            f(x) = f(y)
            f(y) = f(x)
            f(y) != f(x)
            false
        }
        x != y
        x < y
    }
}

/// An order-dual isomorphism pair reverses and reflects strict order.
theorem order_dual_iso_pair_lt_iff_gt[A: PartialOrder, B: PartialOrder](f: A -> B, g: B -> A, x: A, y: A) {
    is_order_dual_iso_pair(f, g) implies (f(x) < f(y) = (x > y))
} by {
    if is_order_dual_iso_pair(f, g) {
        if f(x) < f(y) {
            order_dual_iso_pair_reflects_lt(f, g, y, x)
            y < x
            x > y
        }
        if x > y {
            y < x
            order_dual_iso_pair_apply_lt(f, g, y, x)
            f(x) < f(y)
        }
        f(x) < f(y) = (x > y)
    }
}

/// An order-dual isomorphism pair reverses and reflects reverse non-strict order.
theorem order_dual_iso_pair_ge_iff_le[A: PartialOrder, B: PartialOrder](f: A -> B, g: B -> A, x: A, y: A) {
    is_order_dual_iso_pair(f, g) implies (f(x) >= f(y) = (x <= y))
} by {
    if is_order_dual_iso_pair(f, g) {
        order_dual_iso_pair_le_iff_ge(f, g, y, x)
        f(y) <= f(x) = (y >= x)
        f(x) >= f(y) = (y >= x)
        y >= x = (x <= y)
        f(x) >= f(y) = (x <= y)
    }
}

/// An order-dual isomorphism pair reverses and reflects strict reverse order.
theorem order_dual_iso_pair_gt_iff_lt[A: PartialOrder, B: PartialOrder](f: A -> B, g: B -> A, x: A, y: A) {
    is_order_dual_iso_pair(f, g) implies (f(x) > f(y) = (x < y))
} by {
    if is_order_dual_iso_pair(f, g) {
        order_dual_iso_pair_lt_iff_gt(f, g, y, x)
        f(y) < f(x) = (y > x)
        f(x) > f(y) = (y > x)
        y > x = (x < y)
        f(x) > f(y) = (x < y)
    }
}

/// The forward map of an order-dual isomorphism pair is strictly antitone.
theorem order_dual_iso_pair_map_is_strict_antitone[A: PartialOrder, B: PartialOrder](f: A -> B, g: B -> A) {
    is_order_dual_iso_pair(f, g) implies is_strict_antitone(f)
} by {
    if is_order_dual_iso_pair(f, g) {
        forall(x: A, y: A) {
            if x < y {
                order_dual_iso_pair_apply_lt(f, g, x, y)
                f(y) < f(x)
            }
        }
    }
}

/// The inverse map of an order-dual isomorphism pair is strictly antitone.
theorem order_dual_iso_pair_inv_is_strict_antitone[A: PartialOrder, B: PartialOrder](f: A -> B, g: B -> A) {
    is_order_dual_iso_pair(f, g) implies is_strict_antitone(g)
} by {
    if is_order_dual_iso_pair(f, g) {
        order_dual_iso_pair_swap(f, g)
        order_dual_iso_pair_map_is_strict_antitone(g, f)
        is_strict_antitone(g)
    }
}

/// An order-dual isomorphism with explicit inverse data.
structure OrderDualIso[A: PartialOrder, B: PartialOrder] {
    /// The order-reversing map.
    map: A -> B

    /// The inverse order-reversing map.
    inv: B -> A
} constraint {
    is_order_dual_iso_pair(map, inv)
}

/// Construction of an order-dual isomorphism remembers the underlying map.
theorem order_dual_iso_new_map[A: PartialOrder, B: PartialOrder](f: A -> B, g: B -> A, e: OrderDualIso[A, B]) {
    OrderDualIso[A, B].new(f, g) = Option.some(e) implies e.map = f
}

/// Construction of an order-dual isomorphism remembers the inverse map.
theorem order_dual_iso_new_inv[A: PartialOrder, B: PartialOrder](f: A -> B, g: B -> A, e: OrderDualIso[A, B]) {
    OrderDualIso[A, B].new(f, g) = Option.some(e) implies e.inv = g
}

/// Every order-dual isomorphism is reconstructed from its maps.
theorem order_dual_iso_new_self[A: PartialOrder, B: PartialOrder](e: OrderDualIso[A, B]) {
    OrderDualIso[A, B].new(e.map, e.inv) = Option.some(e)
}

/// Equality of options containing order-dual isomorphisms is equality of order-dual isomorphisms.
theorem order_dual_iso_some_injective[A: PartialOrder, B: PartialOrder](e: OrderDualIso[A, B], h: OrderDualIso[A, B]) {
    Option.some(e) = Option.some(h) implies e = h
}

/// Order-dual isomorphisms are equal when their maps and inverse maps are equal.
theorem order_dual_iso_ext[A: PartialOrder, B: PartialOrder](e: OrderDualIso[A, B], h: OrderDualIso[A, B]) {
    e.map = h.map and e.inv = h.inv implies e = h
} by {
    if e.map = h.map and e.inv = h.inv {
        OrderDualIso[A, B].new(e.map, e.inv) = Option.some(e)
        OrderDualIso[A, B].new(h.map, h.inv) = Option.some(h)
        Option.some(e) = Option.some(h)
        order_dual_iso_some_injective(e, h)
    }
}

/// Equal order-dual isomorphisms have equal underlying maps.
theorem order_dual_iso_eq_map[A: PartialOrder, B: PartialOrder](e: OrderDualIso[A, B], h: OrderDualIso[A, B]) {
    e = h implies e.map = h.map
}

/// Equal order-dual isomorphisms have equal inverse maps.
theorem order_dual_iso_eq_inv[A: PartialOrder, B: PartialOrder](e: OrderDualIso[A, B], h: OrderDualIso[A, B]) {
    e = h implies e.inv = h.inv
}

/// The inverse map is a left inverse of the map.
theorem order_dual_iso_left_inverse[A: PartialOrder, B: PartialOrder](e: OrderDualIso[A, B], a: A) {
    e.inv(e.map(a)) = a
} by {
    OrderDualIso[A, B].constraint(e.map, e.inv)
    is_order_dual_iso_pair(e.map, e.inv)
    order_dual_iso_pair_left_inverse(e.map, e.inv, a)
}

/// The inverse map is a right inverse of the map.
theorem order_dual_iso_right_inverse[A: PartialOrder, B: PartialOrder](e: OrderDualIso[A, B], b: B) {
    e.map(e.inv(b)) = b
} by {
    OrderDualIso[A, B].constraint(e.map, e.inv)
    is_order_dual_iso_pair(e.map, e.inv)
    order_dual_iso_pair_right_inverse(e.map, e.inv, b)
}

/// The underlying map of an order-dual isomorphism is antitone.
theorem order_dual_iso_map_is_antitone[A: PartialOrder, B: PartialOrder](e: OrderDualIso[A, B]) {
    is_antitone(e.map)
} by {
    OrderDualIso[A, B].constraint(e.map, e.inv)
    order_dual_iso_pair_map_is_antitone(e.map, e.inv)
}

/// The inverse map of an order-dual isomorphism is antitone.
theorem order_dual_iso_inv_is_antitone[A: PartialOrder, B: PartialOrder](e: OrderDualIso[A, B]) {
    is_antitone(e.inv)
} by {
    OrderDualIso[A, B].constraint(e.map, e.inv)
    order_dual_iso_pair_inv_is_antitone(e.map, e.inv)
}

/// An order-dual isomorphism reverses non-strict order.
theorem order_dual_iso_apply_le[A: PartialOrder, B: PartialOrder](e: OrderDualIso[A, B], x: A, y: A) {
    x <= y implies e.map(y) <= e.map(x)
} by {
    if x <= y {
        OrderDualIso[A, B].constraint(e.map, e.inv)
        order_dual_iso_pair_apply_le(e.map, e.inv, x, y)
        e.map(y) <= e.map(x)
    }
}

/// An order-dual isomorphism reflects reversed non-strict order.
theorem order_dual_iso_reflects_le[A: PartialOrder, B: PartialOrder](e: OrderDualIso[A, B], x: A, y: A) {
    e.map(y) <= e.map(x) implies x <= y
} by {
    if e.map(y) <= e.map(x) {
        OrderDualIso[A, B].constraint(e.map, e.inv)
        order_dual_iso_pair_reflects_le(e.map, e.inv, x, y)
        x <= y
    }
}

/// An order-dual isomorphism reverses and reflects non-strict order.
theorem order_dual_iso_le_iff_ge[A: PartialOrder, B: PartialOrder](e: OrderDualIso[A, B], x: A, y: A) {
    e.map(x) <= e.map(y) = (x >= y)
} by {
    OrderDualIso[A, B].constraint(e.map, e.inv)
    order_dual_iso_pair_le_iff_ge(e.map, e.inv, x, y)
}

/// An order-dual isomorphism reverses strict order.
theorem order_dual_iso_apply_lt[A: PartialOrder, B: PartialOrder](e: OrderDualIso[A, B], x: A, y: A) {
    x < y implies e.map(y) < e.map(x)
} by {
    if x < y {
        OrderDualIso[A, B].constraint(e.map, e.inv)
        order_dual_iso_pair_apply_lt(e.map, e.inv, x, y)
        e.map(y) < e.map(x)
    }
}

/// An order-dual isomorphism reflects reversed strict order.
theorem order_dual_iso_reflects_lt[A: PartialOrder, B: PartialOrder](e: OrderDualIso[A, B], x: A, y: A) {
    e.map(y) < e.map(x) implies x < y
} by {
    if e.map(y) < e.map(x) {
        OrderDualIso[A, B].constraint(e.map, e.inv)
        order_dual_iso_pair_reflects_lt(e.map, e.inv, x, y)
        x < y
    }
}

/// An order-dual isomorphism reverses and reflects strict order.
theorem order_dual_iso_lt_iff_gt[A: PartialOrder, B: PartialOrder](e: OrderDualIso[A, B], x: A, y: A) {
    e.map(x) < e.map(y) = (x > y)
} by {
    OrderDualIso[A, B].constraint(e.map, e.inv)
    order_dual_iso_pair_lt_iff_gt(e.map, e.inv, x, y)
}

/// An order-dual isomorphism reverses and reflects reverse non-strict order.
theorem order_dual_iso_ge_iff_le[A: PartialOrder, B: PartialOrder](e: OrderDualIso[A, B], x: A, y: A) {
    e.map(x) >= e.map(y) = (x <= y)
} by {
    OrderDualIso[A, B].constraint(e.map, e.inv)
    order_dual_iso_pair_ge_iff_le(e.map, e.inv, x, y)
}

/// An order-dual isomorphism reverses and reflects strict reverse order.
theorem order_dual_iso_gt_iff_lt[A: PartialOrder, B: PartialOrder](e: OrderDualIso[A, B], x: A, y: A) {
    e.map(x) > e.map(y) = (x < y)
} by {
    OrderDualIso[A, B].constraint(e.map, e.inv)
    order_dual_iso_pair_gt_iff_lt(e.map, e.inv, x, y)
}

/// The underlying map of an order-dual isomorphism is strictly antitone.
theorem order_dual_iso_map_is_strict_antitone[A: PartialOrder, B: PartialOrder](e: OrderDualIso[A, B]) {
    is_strict_antitone(e.map)
} by {
    OrderDualIso[A, B].constraint(e.map, e.inv)
    order_dual_iso_pair_map_is_strict_antitone(e.map, e.inv)
}

/// The inverse map of an order-dual isomorphism is strictly antitone.
theorem order_dual_iso_inv_is_strict_antitone[A: PartialOrder, B: PartialOrder](e: OrderDualIso[A, B]) {
    is_strict_antitone(e.inv)
} by {
    OrderDualIso[A, B].constraint(e.map, e.inv)
    order_dual_iso_pair_inv_is_strict_antitone(e.map, e.inv)
}

/// The inverse of an order-dual isomorphism.
let order_dual_iso_inverse[A: PartialOrder, B: PartialOrder](e: OrderDualIso[A, B]) -> result: OrderDualIso[B, A] satisfy {
    OrderDualIso[B, A].new(e.inv, e.map) = Option.some(result)
} by {
    OrderDualIso[A, B].constraint(e.map, e.inv)
    order_dual_iso_pair_swap(e.map, e.inv)
}

/// The inverse order-dual isomorphism has the original inverse map.
theorem order_dual_iso_inverse_map[A: PartialOrder, B: PartialOrder](e: OrderDualIso[A, B]) {
    order_dual_iso_inverse(e).map = e.inv
} by {
    let h = order_dual_iso_inverse(e)
    OrderDualIso[B, A].new(e.inv, e.map) = Option.some(h)
    order_dual_iso_new_map(e.inv, e.map, h)
}

/// The inverse order-dual isomorphism has the original map as inverse.
theorem order_dual_iso_inverse_inv[A: PartialOrder, B: PartialOrder](e: OrderDualIso[A, B]) {
    order_dual_iso_inverse(e).inv = e.map
} by {
    let h = order_dual_iso_inverse(e)
    OrderDualIso[B, A].new(e.inv, e.map) = Option.some(h)
    order_dual_iso_new_inv(e.inv, e.map, h)
}

/// Composing two order-dual isomorphism pairs gives an order isomorphism pair.
theorem order_dual_iso_pair_compose_dual[A: PartialOrder, B: PartialOrder, C: PartialOrder](
    f: B -> C,
    g: C -> B,
    h: A -> B,
    k: B -> A
) {
    is_order_dual_iso_pair(f, g) and is_order_dual_iso_pair(h, k) implies
    is_order_iso_pair(compose(f, h), compose(k, g))
} by {
    if is_order_dual_iso_pair(f, g) and is_order_dual_iso_pair(h, k) {
        let fh = compose(f, h)
        let kg = compose(k, g)
        forall(a: A) {
            kg(fh(a)) = compose(k, g, fh(a))
            compose(k, g, fh(a)) = k(g(fh(a)))
            fh(a) = compose(f, h)(a)
            compose(f, h, a) = f(h(a))
            g(fh(a)) = g(f(h(a)))
            order_dual_iso_pair_left_inverse(f, g, h(a))
            g(f(h(a))) = h(a)
            k(g(fh(a))) = k(h(a))
            order_dual_iso_pair_left_inverse(h, k, a)
            k(h(a)) = a
            kg(fh(a)) = a
        }
        forall(c: C) {
            fh(kg(c)) = compose(f, h, kg(c))
            compose(f, h, kg(c)) = f(h(kg(c)))
            kg(c) = compose(k, g)(c)
            compose(k, g, c) = k(g(c))
            h(kg(c)) = h(k(g(c)))
            order_dual_iso_pair_right_inverse(h, k, g(c))
            h(k(g(c))) = g(c)
            f(h(kg(c))) = f(g(c))
            order_dual_iso_pair_right_inverse(f, g, c)
            f(g(c)) = c
            fh(kg(c)) = c
        }
        forall(x: A, y: A) {
            if fh(x) <= fh(y) {
                fh(x) = f(h(x))
                fh(y) = f(h(y))
                f(h(x)) <= f(h(y))
                order_dual_iso_pair_reflects_le(f, g, h(y), h(x))
                h(y) <= h(x)
                order_dual_iso_pair_reflects_le(h, k, x, y)
                x <= y
            }
            if x <= y {
                order_dual_iso_pair_apply_le(h, k, x, y)
                h(y) <= h(x)
                order_dual_iso_pair_apply_le(f, g, h(y), h(x))
                f(h(x)) <= f(h(y))
                fh(x) = f(h(x))
                fh(y) = f(h(y))
                fh(x) <= fh(y)
            }
            fh(x) <= fh(y) = (x <= y)
        }
        forall(x: C, y: C) {
            if kg(x) <= kg(y) {
                kg(x) = k(g(x))
                kg(y) = k(g(y))
                k(g(x)) <= k(g(y))
                order_dual_iso_pair_swap(h, k)
                order_dual_iso_pair_reflects_le(k, h, g(y), g(x))
                g(y) <= g(x)
                order_dual_iso_pair_swap(f, g)
                order_dual_iso_pair_reflects_le(g, f, x, y)
                x <= y
            }
            if x <= y {
                order_dual_iso_pair_swap(f, g)
                order_dual_iso_pair_apply_le(g, f, x, y)
                g(y) <= g(x)
                order_dual_iso_pair_swap(h, k)
                order_dual_iso_pair_apply_le(k, h, g(y), g(x))
                k(g(x)) <= k(g(y))
                kg(x) = k(g(x))
                kg(y) = k(g(y))
                kg(x) <= kg(y)
            }
            kg(x) <= kg(y) = (x <= y)
        }
        is_order_embedding(fh)
        is_order_embedding(kg)
        is_order_iso_pair(fh, kg) = (forall(a: A) {
            kg(fh(a)) = a
        } and forall(c: C) {
            fh(kg(c)) = c
        } and is_order_embedding(fh) and is_order_embedding(kg))
        forall(a: A) {
            kg(fh(a)) = a
        }
        forall(c: C) {
            fh(kg(c)) = c
        }
        is_order_embedding(fh) and is_order_embedding(kg)
        (forall(c: C) {
            fh(kg(c)) = c
        } and is_order_embedding(fh) and is_order_embedding(kg))
        (forall(a: A) {
            kg(fh(a)) = a
        } and forall(c: C) {
            fh(kg(c)) = c
        } and is_order_embedding(fh) and is_order_embedding(kg))
        is_order_iso_pair(fh, kg)
        fh = compose(f, h)
        kg = compose(k, g)
        is_order_iso_pair(compose(f, h), compose(k, g))
    }
}

/// The image predicate transported by an order isomorphism.
define order_iso_image_predicate[A: PartialOrder, B: PartialOrder](e: OrderIso[A, B], p: A -> Bool, y: B) -> Bool {
    p(e.inv(y))
}

/// The preimage predicate transported by an order isomorphism.
define order_iso_preimage_predicate[A: PartialOrder, B: PartialOrder](e: OrderIso[A, B], q: B -> Bool, x: A) -> Bool {
    q(e.map(x))
}

/// The image predicate transported by an order-dual isomorphism.
define order_dual_iso_image_predicate[A: PartialOrder, B: PartialOrder](
    e: OrderDualIso[A, B],
    p: A -> Bool,
    y: B
) -> Bool {
    p(e.inv(y))
}

/// The preimage predicate transported by an order-dual isomorphism.
define order_dual_iso_preimage_predicate[A: PartialOrder, B: PartialOrder](
    e: OrderDualIso[A, B],
    q: B -> Bool,
    x: A
) -> Bool {
    q(e.map(x))
}

/// Construction of an order isomorphism remembers the underlying map.
theorem order_iso_new_map[A: PartialOrder, B: PartialOrder](f: A -> B, g: B -> A, e: OrderIso[A, B]) {
    OrderIso[A, B].new(f, g) = Option.some(e) implies e.map = f
}

/// Construction of an order isomorphism remembers the inverse map.
theorem order_iso_new_inv[A: PartialOrder, B: PartialOrder](f: A -> B, g: B -> A, e: OrderIso[A, B]) {
    OrderIso[A, B].new(f, g) = Option.some(e) implies e.inv = g
}

/// Every order isomorphism is reconstructed from its maps.
theorem order_iso_new_self[A: PartialOrder, B: PartialOrder](e: OrderIso[A, B]) {
    OrderIso[A, B].new(e.map, e.inv) = Option.some(e)
}

/// Equality of options containing order isomorphisms is equality of order isomorphisms.
theorem order_iso_some_injective[A: PartialOrder, B: PartialOrder](e: OrderIso[A, B], h: OrderIso[A, B]) {
    Option.some(e) = Option.some(h) implies e = h
}

/// Order isomorphisms are equal when their maps and inverse maps are equal.
theorem order_iso_ext[A: PartialOrder, B: PartialOrder](e: OrderIso[A, B], h: OrderIso[A, B]) {
    e.map = h.map and e.inv = h.inv implies e = h
} by {
    if e.map = h.map and e.inv = h.inv {
        OrderIso[A, B].new(e.map, e.inv) = Option.some(e)
        OrderIso[A, B].new(h.map, h.inv) = Option.some(h)
        Option.some(e) = Option.some(h)
        order_iso_some_injective(e, h)
    }
}

/// Equal order isomorphisms have equal underlying maps.
theorem order_iso_eq_map[A: PartialOrder, B: PartialOrder](e: OrderIso[A, B], h: OrderIso[A, B]) {
    e = h implies e.map = h.map
}

/// Equal order isomorphisms have equal inverse maps.
theorem order_iso_eq_inv[A: PartialOrder, B: PartialOrder](e: OrderIso[A, B], h: OrderIso[A, B]) {
    e = h implies e.inv = h.inv
}

/// The inverse map is a left inverse of the map.
theorem order_iso_left_inverse[A: PartialOrder, B: PartialOrder](e: OrderIso[A, B], a: A) {
    e.inv(e.map(a)) = a
} by {
    OrderIso[A, B].constraint(e.map, e.inv)
    is_order_iso_pair(e.map, e.inv)
    order_iso_pair_left_inverse(e.map, e.inv, a)
}

/// The inverse map is a right inverse of the map.
theorem order_iso_right_inverse[A: PartialOrder, B: PartialOrder](e: OrderIso[A, B], b: B) {
    e.map(e.inv(b)) = b
} by {
    OrderIso[A, B].constraint(e.map, e.inv)
    is_order_iso_pair(e.map, e.inv)
    order_iso_pair_right_inverse(e.map, e.inv, b)
}

/// The underlying map of an order isomorphism is an order embedding.
theorem order_iso_map_is_order_embedding[A: PartialOrder, B: PartialOrder](e: OrderIso[A, B]) {
    is_order_embedding(e.map)
} by {
    OrderIso[A, B].constraint(e.map, e.inv)
    is_order_iso_pair(e.map, e.inv)
    order_iso_pair_map_is_order_embedding(e.map, e.inv)
}

/// The inverse map of an order isomorphism is an order embedding.
theorem order_iso_inv_is_order_embedding[A: PartialOrder, B: PartialOrder](e: OrderIso[A, B]) {
    is_order_embedding(e.inv)
} by {
    order_iso_map_is_order_embedding(e)
    forall(x: B, y: B) {
        order_iso_right_inverse(e, x)
        e.map(e.inv(x)) = x
        order_iso_right_inverse(e, y)
        e.map(e.inv(y)) = y
        if e.inv(x) <= e.inv(y) {
            order_embedding_apply_le(e.map, e.inv(x), e.inv(y))
            e.map(e.inv(x)) <= e.map(e.inv(y))
            x <= y
        }
        if x <= y {
            e.map(e.inv(x)) <= e.map(e.inv(y))
            order_embedding_reflects_lte(e.map, e.inv(x), e.inv(y))
            e.inv(x) <= e.inv(y)
        }
        e.inv(x) <= e.inv(y) = (x <= y)
    }
}

/// The underlying map of an order isomorphism is injective.
theorem order_iso_map_is_injective[A: PartialOrder, B: PartialOrder](e: OrderIso[A, B]) {
    is_injective_fn(e.map)
} by {
    forall(x: A, y: A) {
        if e.map(x) = e.map(y) {
            order_iso_left_inverse(e, x)
            order_iso_left_inverse(e, y)
            x = y
        }
    }
}

/// The underlying map of an order isomorphism is surjective.
theorem order_iso_map_is_surjective[A: PartialOrder, B: PartialOrder](e: OrderIso[A, B]) {
    is_surjective_fn(e.map)
} by {
    forall(y: B) {
        exists(x: A) {
            x = e.inv(y) and e.map(x) = y
        }
    }
}

/// The underlying map of an order isomorphism is bijective.
theorem order_iso_map_is_bijection[A: PartialOrder, B: PartialOrder](e: OrderIso[A, B]) {
    is_bijection_fn(e.map)
} by {
    order_iso_map_is_injective(e)
    is_injective_fn(e.map)
    order_iso_map_is_surjective(e)
    is_surjective_fn(e.map)
    is_bijection_fn(e.map)
}

/// The inverse map of an order isomorphism is injective.
theorem order_iso_inv_is_injective[A: PartialOrder, B: PartialOrder](e: OrderIso[A, B]) {
    is_injective_fn(e.inv)
} by {
    forall(x: B, y: B) {
        if e.inv(x) = e.inv(y) {
            order_iso_right_inverse(e, x)
            order_iso_right_inverse(e, y)
            x = y
        }
    }
}

/// The inverse map of an order isomorphism is surjective.
theorem order_iso_inv_is_surjective[A: PartialOrder, B: PartialOrder](e: OrderIso[A, B]) {
    is_surjective_fn(e.inv)
} by {
    forall(x: A) {
        exists(y: B) {
            y = e.map(x) and e.inv(y) = x
        }
    }
}

/// The inverse map of an order isomorphism is bijective.
theorem order_iso_inv_is_bijection[A: PartialOrder, B: PartialOrder](e: OrderIso[A, B]) {
    is_bijection_fn(e.inv)
} by {
    order_iso_inv_is_injective(e)
    is_injective_fn(e.inv)
    order_iso_inv_is_surjective(e)
    is_surjective_fn(e.inv)
    is_bijection_fn(e.inv)
}

/// An order isomorphism preserves non-strict order.
theorem order_iso_apply_le[A: PartialOrder, B: PartialOrder](e: OrderIso[A, B], x: A, y: A) {
    x <= y implies e.map(x) <= e.map(y)
} by {
    if x <= y {
        order_iso_map_is_order_embedding(e)
        order_embedding_apply_le(e.map, x, y)
        e.map(x) <= e.map(y)
    }
}

/// An order isomorphism reflects non-strict order.
theorem order_iso_reflects_le[A: PartialOrder, B: PartialOrder](e: OrderIso[A, B], x: A, y: A) {
    e.map(x) <= e.map(y) implies x <= y
} by {
    if e.map(x) <= e.map(y) {
        order_iso_map_is_order_embedding(e)
        order_embedding_reflects_lte(e.map, x, y)
        x <= y
    }
}

/// An order isomorphism preserves and reflects non-strict order.
theorem order_iso_le_iff_le[A: PartialOrder, B: PartialOrder](e: OrderIso[A, B], x: A, y: A) {
    e.map(x) <= e.map(y) = (x <= y)
} by {
    if e.map(x) <= e.map(y) {
        order_iso_reflects_le(e, x, y)
        x <= y
    }
    if x <= y {
        order_iso_apply_le(e, x, y)
        e.map(x) <= e.map(y)
    }
    e.map(x) <= e.map(y) = (x <= y)
}

/// An inverse order isomorphism preserves non-strict order.
theorem order_iso_inv_apply_le[A: PartialOrder, B: PartialOrder](e: OrderIso[A, B], x: B, y: B) {
    x <= y implies e.inv(x) <= e.inv(y)
} by {
    if x <= y {
        order_iso_inv_is_order_embedding(e)
        order_embedding_apply_le(e.inv, x, y)
        e.inv(x) <= e.inv(y)
    }
}

/// An inverse order isomorphism reflects non-strict order.
theorem order_iso_inv_reflects_le[A: PartialOrder, B: PartialOrder](e: OrderIso[A, B], x: B, y: B) {
    e.inv(x) <= e.inv(y) implies x <= y
} by {
    if e.inv(x) <= e.inv(y) {
        order_iso_inv_is_order_embedding(e)
        order_embedding_reflects_lte(e.inv, x, y)
        x <= y
    }
}

/// An inverse order isomorphism preserves and reflects non-strict order.
theorem order_iso_inv_le_iff_le[A: PartialOrder, B: PartialOrder](e: OrderIso[A, B], x: B, y: B) {
    e.inv(x) <= e.inv(y) = (x <= y)
} by {
    if e.inv(x) <= e.inv(y) {
        order_iso_inv_reflects_le(e, x, y)
        x <= y
    }
    if x <= y {
        order_iso_inv_apply_le(e, x, y)
        e.inv(x) <= e.inv(y)
    }
    e.inv(x) <= e.inv(y) = (x <= y)
}

/// An order isomorphism preserves strict order.
theorem order_iso_apply_lt[A: PartialOrder, B: PartialOrder](e: OrderIso[A, B], x: A, y: A) {
    x < y implies e.map(x) < e.map(y)
} by {
    if x < y {
        order_iso_map_is_order_embedding(e)
        order_embedding_apply_lt(e.map, x, y)
        e.map(x) < e.map(y)
    }
}

/// An order isomorphism reflects strict order.
theorem order_iso_reflects_lt[A: PartialOrder, B: PartialOrder](e: OrderIso[A, B], x: A, y: A) {
    e.map(x) < e.map(y) implies x < y
} by {
    if e.map(x) < e.map(y) {
        order_iso_map_is_order_embedding(e)
        order_embedding_reflects_lt(e.map, x, y)
        x < y
    }
}

/// An order isomorphism preserves and reflects strict order.
theorem order_iso_lt_iff_lt[A: PartialOrder, B: PartialOrder](e: OrderIso[A, B], x: A, y: A) {
    e.map(x) < e.map(y) = (x < y)
} by {
    if e.map(x) < e.map(y) {
        order_iso_reflects_lt(e, x, y)
        x < y
    }
    if x < y {
        order_iso_apply_lt(e, x, y)
        e.map(x) < e.map(y)
    }
    e.map(x) < e.map(y) = (x < y)
}

/// An order isomorphism preserves reverse non-strict order.
theorem order_iso_apply_ge[A: PartialOrder, B: PartialOrder](e: OrderIso[A, B], x: A, y: A) {
    x >= y implies e.map(x) >= e.map(y)
} by {
    if x >= y {
        order_iso_map_is_order_embedding(e)
        order_embedding_apply_ge(e.map, x, y)
        e.map(x) >= e.map(y)
    }
}

/// An order isomorphism reflects reverse non-strict order.
theorem order_iso_reflects_ge[A: PartialOrder, B: PartialOrder](e: OrderIso[A, B], x: A, y: A) {
    e.map(x) >= e.map(y) implies x >= y
} by {
    if e.map(x) >= e.map(y) {
        order_iso_map_is_order_embedding(e)
        order_embedding_reflects_ge(e.map, x, y)
        x >= y
    }
}

/// An order isomorphism preserves and reflects reverse non-strict order.
theorem order_iso_ge_iff_ge[A: PartialOrder, B: PartialOrder](e: OrderIso[A, B], x: A, y: A) {
    e.map(x) >= e.map(y) = (x >= y)
} by {
    if e.map(x) >= e.map(y) {
        order_iso_reflects_ge(e, x, y)
        x >= y
    }
    if x >= y {
        order_iso_apply_ge(e, x, y)
        e.map(x) >= e.map(y)
    }
    e.map(x) >= e.map(y) = (x >= y)
}

/// An order isomorphism preserves strict reverse order.
theorem order_iso_apply_gt[A: PartialOrder, B: PartialOrder](e: OrderIso[A, B], x: A, y: A) {
    x > y implies e.map(x) > e.map(y)
} by {
    if x > y {
        order_iso_map_is_order_embedding(e)
        order_embedding_apply_gt(e.map, x, y)
        e.map(x) > e.map(y)
    }
}

/// An order isomorphism reflects strict reverse order.
theorem order_iso_reflects_gt[A: PartialOrder, B: PartialOrder](e: OrderIso[A, B], x: A, y: A) {
    e.map(x) > e.map(y) implies x > y
} by {
    if e.map(x) > e.map(y) {
        order_iso_map_is_order_embedding(e)
        order_embedding_reflects_gt(e.map, x, y)
        x > y
    }
}

/// An order isomorphism preserves and reflects strict reverse order.
theorem order_iso_gt_iff_gt[A: PartialOrder, B: PartialOrder](e: OrderIso[A, B], x: A, y: A) {
    e.map(x) > e.map(y) = (x > y)
} by {
    if e.map(x) > e.map(y) {
        order_iso_reflects_gt(e, x, y)
        x > y
    }
    if x > y {
        order_iso_apply_gt(e, x, y)
        e.map(x) > e.map(y)
    }
    e.map(x) > e.map(y) = (x > y)
}

/// An inverse order isomorphism preserves strict order.
theorem order_iso_inv_apply_lt[A: PartialOrder, B: PartialOrder](e: OrderIso[A, B], x: B, y: B) {
    x < y implies e.inv(x) < e.inv(y)
} by {
    if x < y {
        order_iso_inv_is_order_embedding(e)
        order_embedding_apply_lt(e.inv, x, y)
        e.inv(x) < e.inv(y)
    }
}

/// An inverse order isomorphism reflects strict order.
theorem order_iso_inv_reflects_lt[A: PartialOrder, B: PartialOrder](e: OrderIso[A, B], x: B, y: B) {
    e.inv(x) < e.inv(y) implies x < y
} by {
    if e.inv(x) < e.inv(y) {
        order_iso_inv_is_order_embedding(e)
        order_embedding_reflects_lt(e.inv, x, y)
        x < y
    }
}

/// An inverse order isomorphism preserves and reflects strict order.
theorem order_iso_inv_lt_iff_lt[A: PartialOrder, B: PartialOrder](e: OrderIso[A, B], x: B, y: B) {
    e.inv(x) < e.inv(y) = (x < y)
} by {
    if e.inv(x) < e.inv(y) {
        order_iso_inv_reflects_lt(e, x, y)
        x < y
    }
    if x < y {
        order_iso_inv_apply_lt(e, x, y)
        e.inv(x) < e.inv(y)
    }
    e.inv(x) < e.inv(y) = (x < y)
}

/// An inverse order isomorphism preserves reverse non-strict order.
theorem order_iso_inv_apply_ge[A: PartialOrder, B: PartialOrder](e: OrderIso[A, B], x: B, y: B) {
    x >= y implies e.inv(x) >= e.inv(y)
} by {
    if x >= y {
        order_iso_inv_is_order_embedding(e)
        order_embedding_apply_ge(e.inv, x, y)
        e.inv(x) >= e.inv(y)
    }
}

/// An inverse order isomorphism reflects reverse non-strict order.
theorem order_iso_inv_reflects_ge[A: PartialOrder, B: PartialOrder](e: OrderIso[A, B], x: B, y: B) {
    e.inv(x) >= e.inv(y) implies x >= y
} by {
    if e.inv(x) >= e.inv(y) {
        order_iso_inv_is_order_embedding(e)
        order_embedding_reflects_ge(e.inv, x, y)
        x >= y
    }
}

/// An inverse order isomorphism preserves and reflects reverse non-strict order.
theorem order_iso_inv_ge_iff_ge[A: PartialOrder, B: PartialOrder](e: OrderIso[A, B], x: B, y: B) {
    e.inv(x) >= e.inv(y) = (x >= y)
} by {
    if e.inv(x) >= e.inv(y) {
        order_iso_inv_reflects_ge(e, x, y)
        x >= y
    }
    if x >= y {
        order_iso_inv_apply_ge(e, x, y)
        e.inv(x) >= e.inv(y)
    }
    e.inv(x) >= e.inv(y) = (x >= y)
}

/// An inverse order isomorphism preserves strict reverse order.
theorem order_iso_inv_apply_gt[A: PartialOrder, B: PartialOrder](e: OrderIso[A, B], x: B, y: B) {
    x > y implies e.inv(x) > e.inv(y)
} by {
    if x > y {
        order_iso_inv_is_order_embedding(e)
        order_embedding_apply_gt(e.inv, x, y)
        e.inv(x) > e.inv(y)
    }
}

/// An inverse order isomorphism reflects strict reverse order.
theorem order_iso_inv_reflects_gt[A: PartialOrder, B: PartialOrder](e: OrderIso[A, B], x: B, y: B) {
    e.inv(x) > e.inv(y) implies x > y
} by {
    if e.inv(x) > e.inv(y) {
        order_iso_inv_is_order_embedding(e)
        order_embedding_reflects_gt(e.inv, x, y)
        x > y
    }
}

/// An inverse order isomorphism preserves and reflects strict reverse order.
theorem order_iso_inv_gt_iff_gt[A: PartialOrder, B: PartialOrder](e: OrderIso[A, B], x: B, y: B) {
    e.inv(x) > e.inv(y) = (x > y)
} by {
    if e.inv(x) > e.inv(y) {
        order_iso_inv_reflects_gt(e, x, y)
        x > y
    }
    if x > y {
        order_iso_inv_apply_gt(e, x, y)
        e.inv(x) > e.inv(y)
    }
    e.inv(x) > e.inv(y) = (x > y)
}

/// The image predicate holds at the image of exactly the original points.
theorem order_iso_image_predicate_map[A: PartialOrder, B: PartialOrder](
    e: OrderIso[A, B],
    p: A -> Bool,
    x: A
) {
    order_iso_image_predicate(e, p, e.map(x)) = p(x)
} by {
    order_iso_left_inverse(e, x)
    e.inv(e.map(x)) = x
    order_iso_image_predicate(e, p, e.map(x)) = p(e.inv(e.map(x)))
    order_iso_image_predicate(e, p, e.map(x)) = p(x)
}

/// The preimage predicate holds at the inverse image of exactly the original points.
theorem order_iso_preimage_predicate_inv[A: PartialOrder, B: PartialOrder](
    e: OrderIso[A, B],
    q: B -> Bool,
    y: B
) {
    order_iso_preimage_predicate(e, q, e.inv(y)) = q(y)
} by {
    order_iso_right_inverse(e, y)
    e.map(e.inv(y)) = y
    order_iso_preimage_predicate(e, q, e.inv(y)) = q(e.map(e.inv(y)))
    order_iso_preimage_predicate(e, q, e.inv(y)) = q(y)
}

/// An order isomorphism carries a lower bound to a lower bound of the image predicate.
theorem order_iso_image_lower_bound[A: LinearOrder, B: LinearOrder](
    e: OrderIso[A, B],
    p: A -> Bool,
    a: A
) {
    is_lower_bound(p, a) implies is_lower_bound(order_iso_image_predicate(e, p), e.map(a))
} by {
    if is_lower_bound(p, a) {
        forall(y: B) {
            if order_iso_image_predicate(e, p, y) {
                p(e.inv(y))
                lower_bound_step(p, a, e.inv(y))
                a <= e.inv(y)
                order_iso_apply_le(e, a, e.inv(y))
                e.map(a) <= e.map(e.inv(y))
                order_iso_right_inverse(e, y)
                e.map(e.inv(y)) = y
                e.map(a) <= y
            }
        }
    }
}

/// An order isomorphism carries an upper bound to an upper bound of the image predicate.
theorem order_iso_image_upper_bound[A: LinearOrder, B: LinearOrder](
    e: OrderIso[A, B],
    p: A -> Bool,
    b: A
) {
    is_upper_bound(p, b) implies is_upper_bound(order_iso_image_predicate(e, p), e.map(b))
} by {
    if is_upper_bound(p, b) {
        forall(y: B) {
            if order_iso_image_predicate(e, p, y) {
                p(e.inv(y))
                upper_bound_step(p, b, e.inv(y))
                e.inv(y) <= b
                order_iso_apply_le(e, e.inv(y), b)
                e.map(e.inv(y)) <= e.map(b)
                order_iso_right_inverse(e, y)
                e.map(e.inv(y)) = y
                y <= e.map(b)
            }
        }
    }
}

/// A lower bound of the image predicate reflects to a lower bound of the original predicate.
theorem order_iso_reflects_image_lower_bound[A: LinearOrder, B: LinearOrder](
    e: OrderIso[A, B],
    p: A -> Bool,
    a: A
) {
    is_lower_bound(order_iso_image_predicate(e, p), e.map(a)) implies is_lower_bound(p, a)
} by {
    if is_lower_bound(order_iso_image_predicate(e, p), e.map(a)) {
        forall(x: A) {
            if p(x) {
                order_iso_image_predicate_map(e, p, x)
                order_iso_image_predicate(e, p, e.map(x))
                lower_bound_step(order_iso_image_predicate(e, p), e.map(a), e.map(x))
                e.map(a) <= e.map(x)
                order_iso_reflects_le(e, a, x)
                a <= x
            }
        }
    }
}

/// An upper bound of the image predicate reflects to an upper bound of the original predicate.
theorem order_iso_reflects_image_upper_bound[A: LinearOrder, B: LinearOrder](
    e: OrderIso[A, B],
    p: A -> Bool,
    b: A
) {
    is_upper_bound(order_iso_image_predicate(e, p), e.map(b)) implies is_upper_bound(p, b)
} by {
    if is_upper_bound(order_iso_image_predicate(e, p), e.map(b)) {
        forall(x: A) {
            if p(x) {
                order_iso_image_predicate_map(e, p, x)
                order_iso_image_predicate(e, p, e.map(x))
                upper_bound_step(order_iso_image_predicate(e, p), e.map(b), e.map(x))
                e.map(x) <= e.map(b)
                order_iso_reflects_le(e, x, b)
                x <= b
            }
        }
    }
}

/// Lower bounds are preserved and reflected by image predicates under an order isomorphism.
theorem order_iso_image_lower_bound_iff[A: LinearOrder, B: LinearOrder](
    e: OrderIso[A, B],
    p: A -> Bool,
    a: A
) {
    is_lower_bound(order_iso_image_predicate(e, p), e.map(a)) = is_lower_bound(p, a)
} by {
    if is_lower_bound(order_iso_image_predicate(e, p), e.map(a)) {
        order_iso_reflects_image_lower_bound(e, p, a)
        is_lower_bound(p, a)
    }
    if is_lower_bound(p, a) {
        order_iso_image_lower_bound(e, p, a)
        is_lower_bound(order_iso_image_predicate(e, p), e.map(a))
    }
    is_lower_bound(order_iso_image_predicate(e, p), e.map(a)) = is_lower_bound(p, a)
}

/// Upper bounds are preserved and reflected by image predicates under an order isomorphism.
theorem order_iso_image_upper_bound_iff[A: LinearOrder, B: LinearOrder](
    e: OrderIso[A, B],
    p: A -> Bool,
    b: A
) {
    is_upper_bound(order_iso_image_predicate(e, p), e.map(b)) = is_upper_bound(p, b)
} by {
    if is_upper_bound(order_iso_image_predicate(e, p), e.map(b)) {
        order_iso_reflects_image_upper_bound(e, p, b)
        is_upper_bound(p, b)
    }
    if is_upper_bound(p, b) {
        order_iso_image_upper_bound(e, p, b)
        is_upper_bound(order_iso_image_predicate(e, p), e.map(b))
    }
    is_upper_bound(order_iso_image_predicate(e, p), e.map(b)) = is_upper_bound(p, b)
}

/// An order isomorphism carries an interval bound to an interval bound of the image predicate.
theorem order_iso_image_bounded_by_interval[A: LinearOrder, B: LinearOrder](
    e: OrderIso[A, B],
    p: A -> Bool,
    a: A,
    b: A
) {
    is_bounded_by_interval(p, a, b) implies
    is_bounded_by_interval(order_iso_image_predicate(e, p), e.map(a), e.map(b))
} by {
    if is_bounded_by_interval(p, a, b) {
        bounded_by_interval_imp_lower_bound(p, a, b)
        is_lower_bound(p, a)
        order_iso_image_lower_bound(e, p, a)
        is_lower_bound(order_iso_image_predicate(e, p), e.map(a))
        bounded_by_interval_imp_upper_bound(p, a, b)
        is_upper_bound(p, b)
        order_iso_image_upper_bound(e, p, b)
        is_upper_bound(order_iso_image_predicate(e, p), e.map(b))
        bounds_imp_bounded_by_interval(order_iso_image_predicate(e, p), e.map(a), e.map(b))
        is_bounded_by_interval(order_iso_image_predicate(e, p), e.map(a), e.map(b))
    }
}

/// Interval bounds of image predicates reflect through an order isomorphism.
theorem order_iso_reflects_image_bounded_by_interval[A: LinearOrder, B: LinearOrder](
    e: OrderIso[A, B],
    p: A -> Bool,
    a: A,
    b: A
) {
    is_bounded_by_interval(order_iso_image_predicate(e, p), e.map(a), e.map(b)) implies
    is_bounded_by_interval(p, a, b)
} by {
    if is_bounded_by_interval(order_iso_image_predicate(e, p), e.map(a), e.map(b)) {
        bounded_by_interval_imp_lower_bound(order_iso_image_predicate(e, p), e.map(a), e.map(b))
        is_lower_bound(order_iso_image_predicate(e, p), e.map(a))
        order_iso_reflects_image_lower_bound(e, p, a)
        is_lower_bound(p, a)
        bounded_by_interval_imp_upper_bound(order_iso_image_predicate(e, p), e.map(a), e.map(b))
        is_upper_bound(order_iso_image_predicate(e, p), e.map(b))
        order_iso_reflects_image_upper_bound(e, p, b)
        is_upper_bound(p, b)
        bounds_imp_bounded_by_interval(p, a, b)
        is_bounded_by_interval(p, a, b)
    }
}

/// Interval bounds are preserved and reflected by image predicates under an order isomorphism.
theorem order_iso_image_bounded_by_interval_iff[A: LinearOrder, B: LinearOrder](
    e: OrderIso[A, B],
    p: A -> Bool,
    a: A,
    b: A
) {
    is_bounded_by_interval(order_iso_image_predicate(e, p), e.map(a), e.map(b)) =
    is_bounded_by_interval(p, a, b)
} by {
    if is_bounded_by_interval(order_iso_image_predicate(e, p), e.map(a), e.map(b)) {
        order_iso_reflects_image_bounded_by_interval(e, p, a, b)
        is_bounded_by_interval(p, a, b)
    }
    if is_bounded_by_interval(p, a, b) {
        order_iso_image_bounded_by_interval(e, p, a, b)
        is_bounded_by_interval(order_iso_image_predicate(e, p), e.map(a), e.map(b))
    }
    is_bounded_by_interval(order_iso_image_predicate(e, p), e.map(a), e.map(b)) =
    is_bounded_by_interval(p, a, b)
}

/// Boundedness below is preserved by image predicates under an order isomorphism.
theorem order_iso_image_bounded_below[A: LinearOrder, B: LinearOrder](
    e: OrderIso[A, B],
    p: A -> Bool
) {
    is_bounded_below(p) implies is_bounded_below(order_iso_image_predicate(e, p))
} by {
    if is_bounded_below(p) {
        let a: A satisfy {
            is_lower_bound(p, a)
        }
        order_iso_image_lower_bound(e, p, a)
        is_lower_bound(order_iso_image_predicate(e, p), e.map(a))
        lower_bound_imp_bounded_below(order_iso_image_predicate(e, p), e.map(a))
        is_bounded_below(order_iso_image_predicate(e, p))
    }
}

/// Boundedness above is preserved by image predicates under an order isomorphism.
theorem order_iso_image_bounded_above[A: LinearOrder, B: LinearOrder](
    e: OrderIso[A, B],
    p: A -> Bool
) {
    is_bounded_above(p) implies is_bounded_above(order_iso_image_predicate(e, p))
} by {
    if is_bounded_above(p) {
        let b: A satisfy {
            is_upper_bound(p, b)
        }
        order_iso_image_upper_bound(e, p, b)
        is_upper_bound(order_iso_image_predicate(e, p), e.map(b))
        upper_bound_imp_bounded_above(order_iso_image_predicate(e, p), e.map(b))
        is_bounded_above(order_iso_image_predicate(e, p))
    }
}

/// Boundedness is preserved by image predicates under an order isomorphism.
theorem order_iso_image_bounded[A: LinearOrder, B: LinearOrder](
    e: OrderIso[A, B],
    p: A -> Bool
) {
    is_bounded(p) implies is_bounded(order_iso_image_predicate(e, p))
} by {
    if is_bounded(p) {
        let (a: A, b: A) satisfy {
            is_bounded_by_interval(p, a, b)
        }
        order_iso_image_bounded_by_interval(e, p, a, b)
        is_bounded_by_interval(order_iso_image_predicate(e, p), e.map(a), e.map(b))
        bounded_by_interval_imp_bounded(order_iso_image_predicate(e, p), e.map(a), e.map(b))
        is_bounded(order_iso_image_predicate(e, p))
    }
}

/// An order isomorphism carries a lower bound backward to a lower bound of the preimage predicate.
theorem order_iso_preimage_lower_bound[A: LinearOrder, B: LinearOrder](
    e: OrderIso[A, B],
    q: B -> Bool,
    a: B
) {
    is_lower_bound(q, a) implies is_lower_bound(order_iso_preimage_predicate(e, q), e.inv(a))
} by {
    if is_lower_bound(q, a) {
        forall(x: A) {
            if order_iso_preimage_predicate(e, q, x) {
                q(e.map(x))
                lower_bound_step(q, a, e.map(x))
                a <= e.map(x)
                order_iso_inv_apply_le(e, a, e.map(x))
                e.inv(a) <= e.inv(e.map(x))
                order_iso_left_inverse(e, x)
                e.inv(e.map(x)) = x
                e.inv(a) <= x
            }
        }
    }
}

/// An order isomorphism carries an upper bound backward to an upper bound of the preimage predicate.
theorem order_iso_preimage_upper_bound[A: LinearOrder, B: LinearOrder](
    e: OrderIso[A, B],
    q: B -> Bool,
    b: B
) {
    is_upper_bound(q, b) implies is_upper_bound(order_iso_preimage_predicate(e, q), e.inv(b))
} by {
    if is_upper_bound(q, b) {
        forall(x: A) {
            if order_iso_preimage_predicate(e, q, x) {
                q(e.map(x))
                upper_bound_step(q, b, e.map(x))
                e.map(x) <= b
                order_iso_inv_apply_le(e, e.map(x), b)
                e.inv(e.map(x)) <= e.inv(b)
                order_iso_left_inverse(e, x)
                e.inv(e.map(x)) = x
                x <= e.inv(b)
            }
        }
    }
}

/// A lower bound of the preimage predicate reflects to a lower bound of the original predicate.
theorem order_iso_reflects_preimage_lower_bound[A: LinearOrder, B: LinearOrder](
    e: OrderIso[A, B],
    q: B -> Bool,
    a: B
) {
    is_lower_bound(order_iso_preimage_predicate(e, q), e.inv(a)) implies is_lower_bound(q, a)
} by {
    if is_lower_bound(order_iso_preimage_predicate(e, q), e.inv(a)) {
        forall(y: B) {
            if q(y) {
                order_iso_preimage_predicate_inv(e, q, y)
                order_iso_preimage_predicate(e, q, e.inv(y))
                lower_bound_step(order_iso_preimage_predicate(e, q), e.inv(a), e.inv(y))
                e.inv(a) <= e.inv(y)
                order_iso_inv_reflects_le(e, a, y)
                a <= y
            }
        }
    }
}

/// An upper bound of the preimage predicate reflects to an upper bound of the original predicate.
theorem order_iso_reflects_preimage_upper_bound[A: LinearOrder, B: LinearOrder](
    e: OrderIso[A, B],
    q: B -> Bool,
    b: B
) {
    is_upper_bound(order_iso_preimage_predicate(e, q), e.inv(b)) implies is_upper_bound(q, b)
} by {
    if is_upper_bound(order_iso_preimage_predicate(e, q), e.inv(b)) {
        forall(y: B) {
            if q(y) {
                order_iso_preimage_predicate_inv(e, q, y)
                order_iso_preimage_predicate(e, q, e.inv(y))
                upper_bound_step(order_iso_preimage_predicate(e, q), e.inv(b), e.inv(y))
                e.inv(y) <= e.inv(b)
                order_iso_inv_reflects_le(e, y, b)
                y <= b
            }
        }
    }
}

/// Lower bounds are preserved and reflected by preimage predicates under an order isomorphism.
theorem order_iso_preimage_lower_bound_iff[A: LinearOrder, B: LinearOrder](
    e: OrderIso[A, B],
    q: B -> Bool,
    a: B
) {
    is_lower_bound(order_iso_preimage_predicate(e, q), e.inv(a)) = is_lower_bound(q, a)
} by {
    if is_lower_bound(order_iso_preimage_predicate(e, q), e.inv(a)) {
        order_iso_reflects_preimage_lower_bound(e, q, a)
        is_lower_bound(q, a)
    }
    if is_lower_bound(q, a) {
        order_iso_preimage_lower_bound(e, q, a)
        is_lower_bound(order_iso_preimage_predicate(e, q), e.inv(a))
    }
    is_lower_bound(order_iso_preimage_predicate(e, q), e.inv(a)) = is_lower_bound(q, a)
}

/// Upper bounds are preserved and reflected by preimage predicates under an order isomorphism.
theorem order_iso_preimage_upper_bound_iff[A: LinearOrder, B: LinearOrder](
    e: OrderIso[A, B],
    q: B -> Bool,
    b: B
) {
    is_upper_bound(order_iso_preimage_predicate(e, q), e.inv(b)) = is_upper_bound(q, b)
} by {
    if is_upper_bound(order_iso_preimage_predicate(e, q), e.inv(b)) {
        order_iso_reflects_preimage_upper_bound(e, q, b)
        is_upper_bound(q, b)
    }
    if is_upper_bound(q, b) {
        order_iso_preimage_upper_bound(e, q, b)
        is_upper_bound(order_iso_preimage_predicate(e, q), e.inv(b))
    }
    is_upper_bound(order_iso_preimage_predicate(e, q), e.inv(b)) = is_upper_bound(q, b)
}

/// An order isomorphism carries an interval bound backward to an interval bound of the preimage predicate.
theorem order_iso_preimage_bounded_by_interval[A: LinearOrder, B: LinearOrder](
    e: OrderIso[A, B],
    q: B -> Bool,
    a: B,
    b: B
) {
    is_bounded_by_interval(q, a, b) implies
    is_bounded_by_interval(order_iso_preimage_predicate(e, q), e.inv(a), e.inv(b))
} by {
    if is_bounded_by_interval(q, a, b) {
        bounded_by_interval_imp_lower_bound(q, a, b)
        is_lower_bound(q, a)
        order_iso_preimage_lower_bound(e, q, a)
        is_lower_bound(order_iso_preimage_predicate(e, q), e.inv(a))
        bounded_by_interval_imp_upper_bound(q, a, b)
        is_upper_bound(q, b)
        order_iso_preimage_upper_bound(e, q, b)
        is_upper_bound(order_iso_preimage_predicate(e, q), e.inv(b))
        bounds_imp_bounded_by_interval(order_iso_preimage_predicate(e, q), e.inv(a), e.inv(b))
        is_bounded_by_interval(order_iso_preimage_predicate(e, q), e.inv(a), e.inv(b))
    }
}

/// Interval bounds of preimage predicates reflect through an order isomorphism.
theorem order_iso_reflects_preimage_bounded_by_interval[A: LinearOrder, B: LinearOrder](
    e: OrderIso[A, B],
    q: B -> Bool,
    a: B,
    b: B
) {
    is_bounded_by_interval(order_iso_preimage_predicate(e, q), e.inv(a), e.inv(b)) implies
    is_bounded_by_interval(q, a, b)
} by {
    if is_bounded_by_interval(order_iso_preimage_predicate(e, q), e.inv(a), e.inv(b)) {
        bounded_by_interval_imp_lower_bound(order_iso_preimage_predicate(e, q), e.inv(a), e.inv(b))
        is_lower_bound(order_iso_preimage_predicate(e, q), e.inv(a))
        order_iso_reflects_preimage_lower_bound(e, q, a)
        is_lower_bound(q, a)
        bounded_by_interval_imp_upper_bound(order_iso_preimage_predicate(e, q), e.inv(a), e.inv(b))
        is_upper_bound(order_iso_preimage_predicate(e, q), e.inv(b))
        order_iso_reflects_preimage_upper_bound(e, q, b)
        is_upper_bound(q, b)
        bounds_imp_bounded_by_interval(q, a, b)
        is_bounded_by_interval(q, a, b)
    }
}

/// Interval bounds are preserved and reflected by preimage predicates under an order isomorphism.
theorem order_iso_preimage_bounded_by_interval_iff[A: LinearOrder, B: LinearOrder](
    e: OrderIso[A, B],
    q: B -> Bool,
    a: B,
    b: B
) {
    is_bounded_by_interval(order_iso_preimage_predicate(e, q), e.inv(a), e.inv(b)) =
    is_bounded_by_interval(q, a, b)
} by {
    if is_bounded_by_interval(order_iso_preimage_predicate(e, q), e.inv(a), e.inv(b)) {
        order_iso_reflects_preimage_bounded_by_interval(e, q, a, b)
        is_bounded_by_interval(q, a, b)
    }
    if is_bounded_by_interval(q, a, b) {
        order_iso_preimage_bounded_by_interval(e, q, a, b)
        is_bounded_by_interval(order_iso_preimage_predicate(e, q), e.inv(a), e.inv(b))
    }
    is_bounded_by_interval(order_iso_preimage_predicate(e, q), e.inv(a), e.inv(b)) =
    is_bounded_by_interval(q, a, b)
}

/// Boundedness below is preserved by preimage predicates under an order isomorphism.
theorem order_iso_preimage_bounded_below[A: LinearOrder, B: LinearOrder](
    e: OrderIso[A, B],
    q: B -> Bool
) {
    is_bounded_below(q) implies is_bounded_below(order_iso_preimage_predicate(e, q))
} by {
    if is_bounded_below(q) {
        let a: B satisfy {
            is_lower_bound(q, a)
        }
        order_iso_preimage_lower_bound(e, q, a)
        is_lower_bound(order_iso_preimage_predicate(e, q), e.inv(a))
        lower_bound_imp_bounded_below(order_iso_preimage_predicate(e, q), e.inv(a))
        is_bounded_below(order_iso_preimage_predicate(e, q))
    }
}

/// Boundedness above is preserved by preimage predicates under an order isomorphism.
theorem order_iso_preimage_bounded_above[A: LinearOrder, B: LinearOrder](
    e: OrderIso[A, B],
    q: B -> Bool
) {
    is_bounded_above(q) implies is_bounded_above(order_iso_preimage_predicate(e, q))
} by {
    if is_bounded_above(q) {
        let b: B satisfy {
            is_upper_bound(q, b)
        }
        order_iso_preimage_upper_bound(e, q, b)
        is_upper_bound(order_iso_preimage_predicate(e, q), e.inv(b))
        upper_bound_imp_bounded_above(order_iso_preimage_predicate(e, q), e.inv(b))
        is_bounded_above(order_iso_preimage_predicate(e, q))
    }
}

/// Boundedness is preserved by preimage predicates under an order isomorphism.
theorem order_iso_preimage_bounded[A: LinearOrder, B: LinearOrder](
    e: OrderIso[A, B],
    q: B -> Bool
) {
    is_bounded(q) implies is_bounded(order_iso_preimage_predicate(e, q))
} by {
    if is_bounded(q) {
        let (a: B, b: B) satisfy {
            is_bounded_by_interval(q, a, b)
        }
        order_iso_preimage_bounded_by_interval(e, q, a, b)
        is_bounded_by_interval(order_iso_preimage_predicate(e, q), e.inv(a), e.inv(b))
        bounded_by_interval_imp_bounded(order_iso_preimage_predicate(e, q), e.inv(a), e.inv(b))
        is_bounded(order_iso_preimage_predicate(e, q))
    }
}

/// The dual image predicate holds at the image of exactly the original points.
theorem order_dual_iso_image_predicate_map[A: PartialOrder, B: PartialOrder](
    e: OrderDualIso[A, B],
    p: A -> Bool,
    x: A
) {
    order_dual_iso_image_predicate(e, p, e.map(x)) = p(x)
} by {
    order_dual_iso_left_inverse(e, x)
    e.inv(e.map(x)) = x
    order_dual_iso_image_predicate(e, p, e.map(x)) = p(e.inv(e.map(x)))
    order_dual_iso_image_predicate(e, p, e.map(x)) = p(x)
}

/// The dual preimage predicate holds at the inverse image of exactly the original points.
theorem order_dual_iso_preimage_predicate_inv[A: PartialOrder, B: PartialOrder](
    e: OrderDualIso[A, B],
    q: B -> Bool,
    y: B
) {
    order_dual_iso_preimage_predicate(e, q, e.inv(y)) = q(y)
} by {
    order_dual_iso_right_inverse(e, y)
    e.map(e.inv(y)) = y
    order_dual_iso_preimage_predicate(e, q, e.inv(y)) = q(e.map(e.inv(y)))
    order_dual_iso_preimage_predicate(e, q, e.inv(y)) = q(y)
}

/// An order-dual isomorphism carries a lower bound to an upper bound of the image predicate.
theorem order_dual_iso_image_lower_bound_to_upper_bound[A: LinearOrder, B: LinearOrder](
    e: OrderDualIso[A, B],
    p: A -> Bool,
    a: A
) {
    is_lower_bound(p, a) implies is_upper_bound(order_dual_iso_image_predicate(e, p), e.map(a))
} by {
    if is_lower_bound(p, a) {
        forall(y: B) {
            if order_dual_iso_image_predicate(e, p, y) {
                p(e.inv(y))
                lower_bound_step(p, a, e.inv(y))
                a <= e.inv(y)
                order_dual_iso_apply_le(e, a, e.inv(y))
                e.map(e.inv(y)) <= e.map(a)
                order_dual_iso_right_inverse(e, y)
                e.map(e.inv(y)) = y
                y <= e.map(a)
            }
        }
    }
}

/// An order-dual isomorphism carries an upper bound to a lower bound of the image predicate.
theorem order_dual_iso_image_upper_bound_to_lower_bound[A: LinearOrder, B: LinearOrder](
    e: OrderDualIso[A, B],
    p: A -> Bool,
    b: A
) {
    is_upper_bound(p, b) implies is_lower_bound(order_dual_iso_image_predicate(e, p), e.map(b))
} by {
    if is_upper_bound(p, b) {
        forall(y: B) {
            if order_dual_iso_image_predicate(e, p, y) {
                p(e.inv(y))
                upper_bound_step(p, b, e.inv(y))
                e.inv(y) <= b
                order_dual_iso_apply_le(e, e.inv(y), b)
                e.map(b) <= e.map(e.inv(y))
                order_dual_iso_right_inverse(e, y)
                e.map(e.inv(y)) = y
                e.map(b) <= y
            }
        }
    }
}

/// An image upper bound under an order-dual isomorphism reflects to a lower bound.
theorem order_dual_iso_reflects_image_upper_bound_to_lower_bound[A: LinearOrder, B: LinearOrder](
    e: OrderDualIso[A, B],
    p: A -> Bool,
    a: A
) {
    is_upper_bound(order_dual_iso_image_predicate(e, p), e.map(a)) implies is_lower_bound(p, a)
} by {
    if is_upper_bound(order_dual_iso_image_predicate(e, p), e.map(a)) {
        forall(x: A) {
            if p(x) {
                order_dual_iso_image_predicate_map(e, p, x)
                order_dual_iso_image_predicate(e, p, e.map(x))
                upper_bound_step(order_dual_iso_image_predicate(e, p), e.map(a), e.map(x))
                e.map(x) <= e.map(a)
                order_dual_iso_reflects_le(e, a, x)
                a <= x
            }
        }
    }
}

/// An image lower bound under an order-dual isomorphism reflects to an upper bound.
theorem order_dual_iso_reflects_image_lower_bound_to_upper_bound[A: LinearOrder, B: LinearOrder](
    e: OrderDualIso[A, B],
    p: A -> Bool,
    b: A
) {
    is_lower_bound(order_dual_iso_image_predicate(e, p), e.map(b)) implies is_upper_bound(p, b)
} by {
    if is_lower_bound(order_dual_iso_image_predicate(e, p), e.map(b)) {
        forall(x: A) {
            if p(x) {
                order_dual_iso_image_predicate_map(e, p, x)
                order_dual_iso_image_predicate(e, p, e.map(x))
                lower_bound_step(order_dual_iso_image_predicate(e, p), e.map(b), e.map(x))
                e.map(b) <= e.map(x)
                order_dual_iso_reflects_le(e, x, b)
                x <= b
            }
        }
    }
}

/// Lower bounds become upper bounds under image predicates for order-dual isomorphisms.
theorem order_dual_iso_image_lower_bound_upper_bound_iff[A: LinearOrder, B: LinearOrder](
    e: OrderDualIso[A, B],
    p: A -> Bool,
    a: A
) {
    is_upper_bound(order_dual_iso_image_predicate(e, p), e.map(a)) = is_lower_bound(p, a)
} by {
    if is_upper_bound(order_dual_iso_image_predicate(e, p), e.map(a)) {
        order_dual_iso_reflects_image_upper_bound_to_lower_bound(e, p, a)
        is_lower_bound(p, a)
    }
    if is_lower_bound(p, a) {
        order_dual_iso_image_lower_bound_to_upper_bound(e, p, a)
        is_upper_bound(order_dual_iso_image_predicate(e, p), e.map(a))
    }
    is_upper_bound(order_dual_iso_image_predicate(e, p), e.map(a)) = is_lower_bound(p, a)
}

/// Upper bounds become lower bounds under image predicates for order-dual isomorphisms.
theorem order_dual_iso_image_upper_bound_lower_bound_iff[A: LinearOrder, B: LinearOrder](
    e: OrderDualIso[A, B],
    p: A -> Bool,
    b: A
) {
    is_lower_bound(order_dual_iso_image_predicate(e, p), e.map(b)) = is_upper_bound(p, b)
} by {
    if is_lower_bound(order_dual_iso_image_predicate(e, p), e.map(b)) {
        order_dual_iso_reflects_image_lower_bound_to_upper_bound(e, p, b)
        is_upper_bound(p, b)
    }
    if is_upper_bound(p, b) {
        order_dual_iso_image_upper_bound_to_lower_bound(e, p, b)
        is_lower_bound(order_dual_iso_image_predicate(e, p), e.map(b))
    }
    is_lower_bound(order_dual_iso_image_predicate(e, p), e.map(b)) = is_upper_bound(p, b)
}

/// An order-dual isomorphism carries interval bounds to reversed interval bounds of the image predicate.
theorem order_dual_iso_image_bounded_by_interval[A: LinearOrder, B: LinearOrder](
    e: OrderDualIso[A, B],
    p: A -> Bool,
    a: A,
    b: A
) {
    is_bounded_by_interval(p, a, b) implies
    is_bounded_by_interval(order_dual_iso_image_predicate(e, p), e.map(b), e.map(a))
} by {
    if is_bounded_by_interval(p, a, b) {
        bounded_by_interval_imp_upper_bound(p, a, b)
        is_upper_bound(p, b)
        order_dual_iso_image_upper_bound_to_lower_bound(e, p, b)
        is_lower_bound(order_dual_iso_image_predicate(e, p), e.map(b))
        bounded_by_interval_imp_lower_bound(p, a, b)
        is_lower_bound(p, a)
        order_dual_iso_image_lower_bound_to_upper_bound(e, p, a)
        is_upper_bound(order_dual_iso_image_predicate(e, p), e.map(a))
        bounds_imp_bounded_by_interval(order_dual_iso_image_predicate(e, p), e.map(b), e.map(a))
        is_bounded_by_interval(order_dual_iso_image_predicate(e, p), e.map(b), e.map(a))
    }
}

/// Reversed interval bounds of image predicates reflect through an order-dual isomorphism.
theorem order_dual_iso_reflects_image_bounded_by_interval[A: LinearOrder, B: LinearOrder](
    e: OrderDualIso[A, B],
    p: A -> Bool,
    a: A,
    b: A
) {
    is_bounded_by_interval(order_dual_iso_image_predicate(e, p), e.map(b), e.map(a)) implies
    is_bounded_by_interval(p, a, b)
} by {
    if is_bounded_by_interval(order_dual_iso_image_predicate(e, p), e.map(b), e.map(a)) {
        bounded_by_interval_imp_upper_bound(order_dual_iso_image_predicate(e, p), e.map(b), e.map(a))
        is_upper_bound(order_dual_iso_image_predicate(e, p), e.map(a))
        order_dual_iso_reflects_image_upper_bound_to_lower_bound(e, p, a)
        is_lower_bound(p, a)
        bounded_by_interval_imp_lower_bound(order_dual_iso_image_predicate(e, p), e.map(b), e.map(a))
        is_lower_bound(order_dual_iso_image_predicate(e, p), e.map(b))
        order_dual_iso_reflects_image_lower_bound_to_upper_bound(e, p, b)
        is_upper_bound(p, b)
        bounds_imp_bounded_by_interval(p, a, b)
        is_bounded_by_interval(p, a, b)
    }
}

/// Interval bounds are preserved and reflected with reversed endpoints under an order-dual isomorphism.
theorem order_dual_iso_image_bounded_by_interval_iff[A: LinearOrder, B: LinearOrder](
    e: OrderDualIso[A, B],
    p: A -> Bool,
    a: A,
    b: A
) {
    is_bounded_by_interval(order_dual_iso_image_predicate(e, p), e.map(b), e.map(a)) =
    is_bounded_by_interval(p, a, b)
} by {
    if is_bounded_by_interval(order_dual_iso_image_predicate(e, p), e.map(b), e.map(a)) {
        order_dual_iso_reflects_image_bounded_by_interval(e, p, a, b)
        is_bounded_by_interval(p, a, b)
    }
    if is_bounded_by_interval(p, a, b) {
        order_dual_iso_image_bounded_by_interval(e, p, a, b)
        is_bounded_by_interval(order_dual_iso_image_predicate(e, p), e.map(b), e.map(a))
    }
    is_bounded_by_interval(order_dual_iso_image_predicate(e, p), e.map(b), e.map(a)) =
    is_bounded_by_interval(p, a, b)
}

/// Boundedness below becomes boundedness above for image predicates under an order-dual isomorphism.
theorem order_dual_iso_image_bounded_below_to_bounded_above[A: LinearOrder, B: LinearOrder](
    e: OrderDualIso[A, B],
    p: A -> Bool
) {
    is_bounded_below(p) implies is_bounded_above(order_dual_iso_image_predicate(e, p))
} by {
    if is_bounded_below(p) {
        let a: A satisfy {
            is_lower_bound(p, a)
        }
        order_dual_iso_image_lower_bound_to_upper_bound(e, p, a)
        is_upper_bound(order_dual_iso_image_predicate(e, p), e.map(a))
        upper_bound_imp_bounded_above(order_dual_iso_image_predicate(e, p), e.map(a))
        is_bounded_above(order_dual_iso_image_predicate(e, p))
    }
}

/// Boundedness above becomes boundedness below for image predicates under an order-dual isomorphism.
theorem order_dual_iso_image_bounded_above_to_bounded_below[A: LinearOrder, B: LinearOrder](
    e: OrderDualIso[A, B],
    p: A -> Bool
) {
    is_bounded_above(p) implies is_bounded_below(order_dual_iso_image_predicate(e, p))
} by {
    if is_bounded_above(p) {
        let b: A satisfy {
            is_upper_bound(p, b)
        }
        order_dual_iso_image_upper_bound_to_lower_bound(e, p, b)
        is_lower_bound(order_dual_iso_image_predicate(e, p), e.map(b))
        lower_bound_imp_bounded_below(order_dual_iso_image_predicate(e, p), e.map(b))
        is_bounded_below(order_dual_iso_image_predicate(e, p))
    }
}

/// Boundedness is preserved by image predicates under an order-dual isomorphism.
theorem order_dual_iso_image_bounded[A: LinearOrder, B: LinearOrder](
    e: OrderDualIso[A, B],
    p: A -> Bool
) {
    is_bounded(p) implies is_bounded(order_dual_iso_image_predicate(e, p))
} by {
    if is_bounded(p) {
        let (a: A, b: A) satisfy {
            is_bounded_by_interval(p, a, b)
        }
        order_dual_iso_image_bounded_by_interval(e, p, a, b)
        is_bounded_by_interval(order_dual_iso_image_predicate(e, p), e.map(b), e.map(a))
        bounded_by_interval_imp_bounded(order_dual_iso_image_predicate(e, p), e.map(b), e.map(a))
        is_bounded(order_dual_iso_image_predicate(e, p))
    }
}

/// An order-dual isomorphism carries a lower bound backward to an upper bound of the preimage predicate.
theorem order_dual_iso_preimage_lower_bound_to_upper_bound[A: LinearOrder, B: LinearOrder](
    e: OrderDualIso[A, B],
    q: B -> Bool,
    a: B
) {
    is_lower_bound(q, a) implies is_upper_bound(order_dual_iso_preimage_predicate(e, q), e.inv(a))
} by {
    if is_lower_bound(q, a) {
        forall(x: A) {
            if order_dual_iso_preimage_predicate(e, q, x) {
                q(e.map(x))
                lower_bound_step(q, a, e.map(x))
                a <= e.map(x)
                order_dual_iso_apply_le(order_dual_iso_inverse(e), a, e.map(x))
                e.inv(e.map(x)) <= e.inv(a)
                order_dual_iso_left_inverse(e, x)
                e.inv(e.map(x)) = x
                x <= e.inv(a)
            }
        }
    }
}

/// An order-dual isomorphism carries an upper bound backward to a lower bound of the preimage predicate.
theorem order_dual_iso_preimage_upper_bound_to_lower_bound[A: LinearOrder, B: LinearOrder](
    e: OrderDualIso[A, B],
    q: B -> Bool,
    b: B
) {
    is_upper_bound(q, b) implies is_lower_bound(order_dual_iso_preimage_predicate(e, q), e.inv(b))
} by {
    if is_upper_bound(q, b) {
        forall(x: A) {
            if order_dual_iso_preimage_predicate(e, q, x) {
                q(e.map(x))
                upper_bound_step(q, b, e.map(x))
                e.map(x) <= b
                order_dual_iso_apply_le(order_dual_iso_inverse(e), e.map(x), b)
                e.inv(b) <= e.inv(e.map(x))
                order_dual_iso_left_inverse(e, x)
                e.inv(e.map(x)) = x
                e.inv(b) <= x
            }
        }
    }
}

/// A preimage upper bound under an order-dual isomorphism reflects to a lower bound.
theorem order_dual_iso_reflects_preimage_upper_bound_to_lower_bound[A: LinearOrder, B: LinearOrder](
    e: OrderDualIso[A, B],
    q: B -> Bool,
    a: B
) {
    is_upper_bound(order_dual_iso_preimage_predicate(e, q), e.inv(a)) implies is_lower_bound(q, a)
} by {
    if is_upper_bound(order_dual_iso_preimage_predicate(e, q), e.inv(a)) {
        forall(y: B) {
            if q(y) {
                order_dual_iso_preimage_predicate_inv(e, q, y)
                order_dual_iso_preimage_predicate(e, q, e.inv(y))
                upper_bound_step(order_dual_iso_preimage_predicate(e, q), e.inv(a), e.inv(y))
                e.inv(y) <= e.inv(a)
                order_dual_iso_reflects_le(order_dual_iso_inverse(e), a, y)
                a <= y
            }
        }
    }
}

/// A preimage lower bound under an order-dual isomorphism reflects to an upper bound.
theorem order_dual_iso_reflects_preimage_lower_bound_to_upper_bound[A: LinearOrder, B: LinearOrder](
    e: OrderDualIso[A, B],
    q: B -> Bool,
    b: B
) {
    is_lower_bound(order_dual_iso_preimage_predicate(e, q), e.inv(b)) implies is_upper_bound(q, b)
} by {
    if is_lower_bound(order_dual_iso_preimage_predicate(e, q), e.inv(b)) {
        forall(y: B) {
            if q(y) {
                order_dual_iso_preimage_predicate_inv(e, q, y)
                order_dual_iso_preimage_predicate(e, q, e.inv(y))
                lower_bound_step(order_dual_iso_preimage_predicate(e, q), e.inv(b), e.inv(y))
                e.inv(b) <= e.inv(y)
                order_dual_iso_reflects_le(order_dual_iso_inverse(e), y, b)
                y <= b
            }
        }
    }
}

/// Lower bounds become upper bounds under preimage predicates for order-dual isomorphisms.
theorem order_dual_iso_preimage_lower_bound_upper_bound_iff[A: LinearOrder, B: LinearOrder](
    e: OrderDualIso[A, B],
    q: B -> Bool,
    a: B
) {
    is_upper_bound(order_dual_iso_preimage_predicate(e, q), e.inv(a)) = is_lower_bound(q, a)
} by {
    if is_upper_bound(order_dual_iso_preimage_predicate(e, q), e.inv(a)) {
        order_dual_iso_reflects_preimage_upper_bound_to_lower_bound(e, q, a)
        is_lower_bound(q, a)
    }
    if is_lower_bound(q, a) {
        order_dual_iso_preimage_lower_bound_to_upper_bound(e, q, a)
        is_upper_bound(order_dual_iso_preimage_predicate(e, q), e.inv(a))
    }
    is_upper_bound(order_dual_iso_preimage_predicate(e, q), e.inv(a)) = is_lower_bound(q, a)
}

/// Upper bounds become lower bounds under preimage predicates for order-dual isomorphisms.
theorem order_dual_iso_preimage_upper_bound_lower_bound_iff[A: LinearOrder, B: LinearOrder](
    e: OrderDualIso[A, B],
    q: B -> Bool,
    b: B
) {
    is_lower_bound(order_dual_iso_preimage_predicate(e, q), e.inv(b)) = is_upper_bound(q, b)
} by {
    if is_lower_bound(order_dual_iso_preimage_predicate(e, q), e.inv(b)) {
        order_dual_iso_reflects_preimage_lower_bound_to_upper_bound(e, q, b)
        is_upper_bound(q, b)
    }
    if is_upper_bound(q, b) {
        order_dual_iso_preimage_upper_bound_to_lower_bound(e, q, b)
        is_lower_bound(order_dual_iso_preimage_predicate(e, q), e.inv(b))
    }
    is_lower_bound(order_dual_iso_preimage_predicate(e, q), e.inv(b)) = is_upper_bound(q, b)
}

/// An order-dual isomorphism carries interval bounds backward to reversed interval bounds of the preimage predicate.
theorem order_dual_iso_preimage_bounded_by_interval[A: LinearOrder, B: LinearOrder](
    e: OrderDualIso[A, B],
    q: B -> Bool,
    a: B,
    b: B
) {
    is_bounded_by_interval(q, a, b) implies
    is_bounded_by_interval(order_dual_iso_preimage_predicate(e, q), e.inv(b), e.inv(a))
} by {
    if is_bounded_by_interval(q, a, b) {
        bounded_by_interval_imp_upper_bound(q, a, b)
        is_upper_bound(q, b)
        order_dual_iso_preimage_upper_bound_to_lower_bound(e, q, b)
        is_lower_bound(order_dual_iso_preimage_predicate(e, q), e.inv(b))
        bounded_by_interval_imp_lower_bound(q, a, b)
        is_lower_bound(q, a)
        order_dual_iso_preimage_lower_bound_to_upper_bound(e, q, a)
        is_upper_bound(order_dual_iso_preimage_predicate(e, q), e.inv(a))
        bounds_imp_bounded_by_interval(order_dual_iso_preimage_predicate(e, q), e.inv(b), e.inv(a))
        is_bounded_by_interval(order_dual_iso_preimage_predicate(e, q), e.inv(b), e.inv(a))
    }
}

/// Reversed interval bounds of preimage predicates reflect through an order-dual isomorphism.
theorem order_dual_iso_reflects_preimage_bounded_by_interval[A: LinearOrder, B: LinearOrder](
    e: OrderDualIso[A, B],
    q: B -> Bool,
    a: B,
    b: B
) {
    is_bounded_by_interval(order_dual_iso_preimage_predicate(e, q), e.inv(b), e.inv(a)) implies
    is_bounded_by_interval(q, a, b)
} by {
    if is_bounded_by_interval(order_dual_iso_preimage_predicate(e, q), e.inv(b), e.inv(a)) {
        bounded_by_interval_imp_upper_bound(order_dual_iso_preimage_predicate(e, q), e.inv(b), e.inv(a))
        is_upper_bound(order_dual_iso_preimage_predicate(e, q), e.inv(a))
        order_dual_iso_reflects_preimage_upper_bound_to_lower_bound(e, q, a)
        is_lower_bound(q, a)
        bounded_by_interval_imp_lower_bound(order_dual_iso_preimage_predicate(e, q), e.inv(b), e.inv(a))
        is_lower_bound(order_dual_iso_preimage_predicate(e, q), e.inv(b))
        order_dual_iso_reflects_preimage_lower_bound_to_upper_bound(e, q, b)
        is_upper_bound(q, b)
        bounds_imp_bounded_by_interval(q, a, b)
        is_bounded_by_interval(q, a, b)
    }
}

/// Interval bounds are preserved and reflected by preimage predicates with reversed endpoints under an order-dual isomorphism.
theorem order_dual_iso_preimage_bounded_by_interval_iff[A: LinearOrder, B: LinearOrder](
    e: OrderDualIso[A, B],
    q: B -> Bool,
    a: B,
    b: B
) {
    is_bounded_by_interval(order_dual_iso_preimage_predicate(e, q), e.inv(b), e.inv(a)) =
    is_bounded_by_interval(q, a, b)
} by {
    if is_bounded_by_interval(order_dual_iso_preimage_predicate(e, q), e.inv(b), e.inv(a)) {
        order_dual_iso_reflects_preimage_bounded_by_interval(e, q, a, b)
        is_bounded_by_interval(q, a, b)
    }
    if is_bounded_by_interval(q, a, b) {
        order_dual_iso_preimage_bounded_by_interval(e, q, a, b)
        is_bounded_by_interval(order_dual_iso_preimage_predicate(e, q), e.inv(b), e.inv(a))
    }
    is_bounded_by_interval(order_dual_iso_preimage_predicate(e, q), e.inv(b), e.inv(a)) =
    is_bounded_by_interval(q, a, b)
}

/// Boundedness below becomes boundedness above for preimage predicates under an order-dual isomorphism.
theorem order_dual_iso_preimage_bounded_below_to_bounded_above[A: LinearOrder, B: LinearOrder](
    e: OrderDualIso[A, B],
    q: B -> Bool
) {
    is_bounded_below(q) implies is_bounded_above(order_dual_iso_preimage_predicate(e, q))
} by {
    if is_bounded_below(q) {
        let a: B satisfy {
            is_lower_bound(q, a)
        }
        order_dual_iso_preimage_lower_bound_to_upper_bound(e, q, a)
        is_upper_bound(order_dual_iso_preimage_predicate(e, q), e.inv(a))
        upper_bound_imp_bounded_above(order_dual_iso_preimage_predicate(e, q), e.inv(a))
        is_bounded_above(order_dual_iso_preimage_predicate(e, q))
    }
}

/// Boundedness above becomes boundedness below for preimage predicates under an order-dual isomorphism.
theorem order_dual_iso_preimage_bounded_above_to_bounded_below[A: LinearOrder, B: LinearOrder](
    e: OrderDualIso[A, B],
    q: B -> Bool
) {
    is_bounded_above(q) implies is_bounded_below(order_dual_iso_preimage_predicate(e, q))
} by {
    if is_bounded_above(q) {
        let b: B satisfy {
            is_upper_bound(q, b)
        }
        order_dual_iso_preimage_upper_bound_to_lower_bound(e, q, b)
        is_lower_bound(order_dual_iso_preimage_predicate(e, q), e.inv(b))
        lower_bound_imp_bounded_below(order_dual_iso_preimage_predicate(e, q), e.inv(b))
        is_bounded_below(order_dual_iso_preimage_predicate(e, q))
    }
}

/// Boundedness is preserved by preimage predicates under an order-dual isomorphism.
theorem order_dual_iso_preimage_bounded[A: LinearOrder, B: LinearOrder](
    e: OrderDualIso[A, B],
    q: B -> Bool
) {
    is_bounded(q) implies is_bounded(order_dual_iso_preimage_predicate(e, q))
} by {
    if is_bounded(q) {
        let (a: B, b: B) satisfy {
            is_bounded_by_interval(q, a, b)
        }
        order_dual_iso_preimage_bounded_by_interval(e, q, a, b)
        is_bounded_by_interval(order_dual_iso_preimage_predicate(e, q), e.inv(b), e.inv(a))
        bounded_by_interval_imp_bounded(order_dual_iso_preimage_predicate(e, q), e.inv(b), e.inv(a))
        is_bounded(order_dual_iso_preimage_predicate(e, q))
    }
}

/// The identity map as an order isomorphism.
let identity_order_iso[A: PartialOrder](anchor: A) -> result: OrderIso[A, A] satisfy {
    result.map = identity_fn[A] and result.inv = identity_fn[A]
} by {
    identity_is_order_embedding[A]
    is_order_embedding(identity_fn[A])
    forall(a: A) {
        identity_fn[A](identity_fn[A](a)) = a
    }
    is_order_iso_pair(identity_fn[A], identity_fn[A])
    let e: OrderIso[A, A] satisfy {
        OrderIso[A, A].new(identity_fn[A], identity_fn[A]) = Option.some(e)
    }
    order_iso_new_map(identity_fn[A], identity_fn[A], e)
    e.map = identity_fn[A]
    order_iso_new_inv(identity_fn[A], identity_fn[A], e)
    e.inv = identity_fn[A]
}

/// The identity order isomorphism has the identity map.
theorem identity_order_iso_map[A: PartialOrder](anchor: A) {
    identity_order_iso(anchor).map = identity_fn[A]
}

/// The identity order isomorphism has the identity inverse map.
theorem identity_order_iso_inv[A: PartialOrder](anchor: A) {
    identity_order_iso(anchor).inv = identity_fn[A]
}

/// The inverse order isomorphism.
let inverse_order_iso[A: PartialOrder, B: PartialOrder](e: OrderIso[A, B]) -> result: OrderIso[B, A] satisfy {
    result.map = e.inv and result.inv = e.map
} by {
    order_iso_inv_is_order_embedding(e)
    is_order_embedding(e.inv)
    order_iso_map_is_order_embedding(e)
    is_order_embedding(e.map)
    forall(b: B) {
        order_iso_right_inverse(e, b)
        e.map(e.inv(b)) = b
    }
    forall(a: A) {
        order_iso_left_inverse(e, a)
        e.inv(e.map(a)) = a
    }
    OrderIso[A, B].constraint(e.map, e.inv)
    is_order_iso_pair(e.map, e.inv)
    order_iso_pair_swap(e.map, e.inv)
    is_order_iso_pair(e.inv, e.map)
    let h: OrderIso[B, A] satisfy {
        OrderIso[B, A].new(e.inv, e.map) = Option.some(h)
    }
    order_iso_new_map(e.inv, e.map, h)
    h.map = e.inv
    order_iso_new_inv(e.inv, e.map, h)
    h.inv = e.map
}

/// The inverse order isomorphism has the inverse map as its map.
theorem inverse_order_iso_map[A: PartialOrder, B: PartialOrder](e: OrderIso[A, B]) {
    inverse_order_iso(e).map = e.inv
}

/// The inverse order isomorphism has the original map as its inverse map.
theorem inverse_order_iso_inv[A: PartialOrder, B: PartialOrder](e: OrderIso[A, B]) {
    inverse_order_iso(e).inv = e.map
}

/// The composite of order isomorphisms.
let compose_order_iso[A: PartialOrder, B: PartialOrder, C: PartialOrder](e: OrderIso[B, C], h: OrderIso[A, B]) -> result: OrderIso[A, C] satisfy {
    result.map = compose(e.map, h.map) and result.inv = compose(h.inv, e.inv)
} by {
    order_iso_map_is_order_embedding(e)
    is_order_embedding(e.map)
    order_iso_map_is_order_embedding(h)
    is_order_embedding(h.map)
    order_embedding_compose(h.map, e.map)
    is_order_embedding(compose(e.map, h.map))
    order_iso_inv_is_order_embedding(e)
    is_order_embedding(e.inv)
    order_iso_inv_is_order_embedding(h)
    is_order_embedding(h.inv)
    order_embedding_compose(e.inv, h.inv)
    is_order_embedding(compose(h.inv, e.inv))
    forall(a: A) {
        compose(h.inv, e.inv, compose(e.map, h.map)(a)) =
            h.inv(e.inv(compose(e.map, h.map)(a)))
        compose(e.map, h.map, a) = e.map(h.map(a))
        e.inv(compose(e.map, h.map)(a)) = e.inv(e.map(h.map(a)))
        order_iso_left_inverse(e, h.map(a))
        e.inv(e.map(h.map(a))) = h.map(a)
        h.inv(e.inv(compose(e.map, h.map)(a))) = h.inv(h.map(a))
        order_iso_left_inverse(h, a)
        h.inv(h.map(a)) = a
        compose(h.inv, e.inv)(compose(e.map, h.map)(a)) = a
    }
    forall(c: C) {
        compose(e.map, h.map, compose(h.inv, e.inv)(c)) =
            e.map(h.map(compose(h.inv, e.inv)(c)))
        compose(h.inv, e.inv, c) = h.inv(e.inv(c))
        h.map(compose(h.inv, e.inv)(c)) = h.map(h.inv(e.inv(c)))
        order_iso_right_inverse(h, e.inv(c))
        h.map(h.inv(e.inv(c))) = e.inv(c)
        e.map(h.map(compose(h.inv, e.inv)(c))) = e.map(e.inv(c))
        order_iso_right_inverse(e, c)
        e.map(e.inv(c)) = c
        compose(e.map, h.map)(compose(h.inv, e.inv)(c)) = c
    }
    OrderIso[B, C].constraint(e.map, e.inv)
    is_order_iso_pair(e.map, e.inv)
    OrderIso[A, B].constraint(h.map, h.inv)
    is_order_iso_pair(h.map, h.inv)
    order_iso_pair_compose(e.map, e.inv, h.map, h.inv)
    is_order_iso_pair(compose(e.map, h.map), compose(h.inv, e.inv))
    let k: OrderIso[A, C] satisfy {
        OrderIso[A, C].new(compose(e.map, h.map), compose(h.inv, e.inv)) = Option.some(k)
    }
    order_iso_new_map(compose(e.map, h.map), compose(h.inv, e.inv), k)
    k.map = compose(e.map, h.map)
    order_iso_new_inv(compose(e.map, h.map), compose(h.inv, e.inv), k)
    k.inv = compose(h.inv, e.inv)
}

/// The composite order isomorphism has the composite map.
theorem compose_order_iso_map[A: PartialOrder, B: PartialOrder, C: PartialOrder](
    e: OrderIso[B, C],
    h: OrderIso[A, B]
) {
    compose_order_iso(e, h).map = compose(e.map, h.map)
}

/// The composite order isomorphism has the composite inverse map.
theorem compose_order_iso_inv[A: PartialOrder, B: PartialOrder, C: PartialOrder](
    e: OrderIso[B, C],
    h: OrderIso[A, B]
) {
    compose_order_iso(e, h).inv = compose(h.inv, e.inv)
}

/// The composite of two order-dual isomorphisms is an order isomorphism.
let compose_order_dual_iso[A: PartialOrder, B: PartialOrder, C: PartialOrder](e: OrderDualIso[B, C], h: OrderDualIso[A, B]) -> result: OrderIso[A, C] satisfy {
    result.map = compose(e.map, h.map) and result.inv = compose(h.inv, e.inv)
} by {
    OrderDualIso[B, C].constraint(e.map, e.inv)
    is_order_dual_iso_pair(e.map, e.inv)
    OrderDualIso[A, B].constraint(h.map, h.inv)
    is_order_dual_iso_pair(h.map, h.inv)
    order_dual_iso_pair_compose_dual(e.map, e.inv, h.map, h.inv)
    is_order_iso_pair(compose(e.map, h.map), compose(h.inv, e.inv))
    let k: OrderIso[A, C] satisfy {
        OrderIso[A, C].new(compose(e.map, h.map), compose(h.inv, e.inv)) = Option.some(k)
    }
    order_iso_new_map(compose(e.map, h.map), compose(h.inv, e.inv), k)
    k.map = compose(e.map, h.map)
    order_iso_new_inv(compose(e.map, h.map), compose(h.inv, e.inv), k)
    k.inv = compose(h.inv, e.inv)
}

/// The composite order isomorphism of two order-dual isomorphisms has the composite map.
theorem compose_order_dual_iso_map[A: PartialOrder, B: PartialOrder, C: PartialOrder](
    e: OrderDualIso[B, C],
    h: OrderDualIso[A, B]
) {
    compose_order_dual_iso(e, h).map = compose(e.map, h.map)
}

/// The composite order isomorphism of two order-dual isomorphisms has the composite inverse map.
theorem compose_order_dual_iso_inv[A: PartialOrder, B: PartialOrder, C: PartialOrder](
    e: OrderDualIso[B, C],
    h: OrderDualIso[A, B]
) {
    compose_order_dual_iso(e, h).inv = compose(h.inv, e.inv)
}

/// Composing two order-dual isomorphisms applies the outer map to the inner map.
theorem compose_order_dual_iso_apply[A: PartialOrder, B: PartialOrder, C: PartialOrder](
    e: OrderDualIso[B, C],
    h: OrderDualIso[A, B],
    a: A
) {
    compose_order_dual_iso(e, h).map(a) = e.map(h.map(a))
} by {
    compose_order_dual_iso_map(e, h)
    compose_order_dual_iso(e, h).map(a) = compose(e.map, h.map)(a)
}

/// An order isomorphism sends closed-interval membership to closed-interval membership.
theorem order_iso_closed_interval_image[A: LinearOrder, B: LinearOrder](e: OrderIso[A, B], a: A, b: A, x: A) {
    closed_interval(a, b, x) implies closed_interval(e.map(a), e.map(b), e.map(x))
} by {
    if closed_interval(a, b, x) {
        order_iso_map_is_order_embedding(e)
        is_order_embedding(e.map)
        order_embedding_closed_interval_image(e.map, a, b, x)
        closed_interval(e.map(a), e.map(b), e.map(x))
    }
}

/// An order isomorphism reflects closed-interval membership.
theorem order_iso_closed_interval_preimage[A: LinearOrder, B: LinearOrder](e: OrderIso[A, B], a: A, b: A, x: A) {
    closed_interval(e.map(a), e.map(b), e.map(x)) implies closed_interval(a, b, x)
} by {
    if closed_interval(e.map(a), e.map(b), e.map(x)) {
        order_iso_map_is_order_embedding(e)
        is_order_embedding(e.map)
        order_embedding_closed_interval_preimage(e.map, a, b, x)
        closed_interval(a, b, x)
    }
}

/// An order isomorphism preserves and reflects closed-interval membership.
theorem order_iso_closed_interval_iff[A: LinearOrder, B: LinearOrder](e: OrderIso[A, B], a: A, b: A, x: A) {
    closed_interval(e.map(a), e.map(b), e.map(x)) = closed_interval(a, b, x)
} by {
    order_iso_map_is_order_embedding(e)
    is_order_embedding(e.map)
    order_embedding_closed_interval_iff(e.map, a, b, x)
    closed_interval(e.map(a), e.map(b), e.map(x)) = closed_interval(a, b, x)
}

/// An order isomorphism sends open-interval membership to open-interval membership.
theorem order_iso_open_interval_image[A: LinearOrder, B: LinearOrder](e: OrderIso[A, B], a: A, b: A, x: A) {
    open_interval(a, b, x) implies open_interval(e.map(a), e.map(b), e.map(x))
} by {
    if open_interval(a, b, x) {
        order_iso_map_is_order_embedding(e)
        is_order_embedding(e.map)
        order_embedding_open_interval_image(e.map, a, b, x)
        open_interval(e.map(a), e.map(b), e.map(x))
    }
}

/// An order isomorphism reflects open-interval membership.
theorem order_iso_open_interval_preimage[A: LinearOrder, B: LinearOrder](e: OrderIso[A, B], a: A, b: A, x: A) {
    open_interval(e.map(a), e.map(b), e.map(x)) implies open_interval(a, b, x)
} by {
    if open_interval(e.map(a), e.map(b), e.map(x)) {
        order_iso_map_is_order_embedding(e)
        is_order_embedding(e.map)
        order_embedding_open_interval_preimage(e.map, a, b, x)
        open_interval(a, b, x)
    }
}

/// An order isomorphism preserves and reflects open-interval membership.
theorem order_iso_open_interval_iff[A: LinearOrder, B: LinearOrder](e: OrderIso[A, B], a: A, b: A, x: A) {
    open_interval(e.map(a), e.map(b), e.map(x)) = open_interval(a, b, x)
} by {
    order_iso_map_is_order_embedding(e)
    is_order_embedding(e.map)
    order_embedding_open_interval_iff(e.map, a, b, x)
    open_interval(e.map(a), e.map(b), e.map(x)) = open_interval(a, b, x)
}

/// An order isomorphism sends left-open interval membership to left-open interval membership.
theorem order_iso_left_open_interval_image[A: LinearOrder, B: LinearOrder](e: OrderIso[A, B], a: A, b: A, x: A) {
    left_open_interval(a, b, x) implies left_open_interval(e.map(a), e.map(b), e.map(x))
} by {
    if left_open_interval(a, b, x) {
        order_iso_map_is_order_embedding(e)
        is_order_embedding(e.map)
        order_embedding_left_open_interval_image(e.map, a, b, x)
        left_open_interval(e.map(a), e.map(b), e.map(x))
    }
}

/// An order isomorphism reflects left-open interval membership.
theorem order_iso_left_open_interval_preimage[A: LinearOrder, B: LinearOrder](e: OrderIso[A, B], a: A, b: A, x: A) {
    left_open_interval(e.map(a), e.map(b), e.map(x)) implies left_open_interval(a, b, x)
} by {
    if left_open_interval(e.map(a), e.map(b), e.map(x)) {
        order_iso_map_is_order_embedding(e)
        is_order_embedding(e.map)
        order_embedding_left_open_interval_preimage(e.map, a, b, x)
        left_open_interval(a, b, x)
    }
}

/// An order isomorphism preserves and reflects left-open interval membership.
theorem order_iso_left_open_interval_iff[A: LinearOrder, B: LinearOrder](e: OrderIso[A, B], a: A, b: A, x: A) {
    left_open_interval(e.map(a), e.map(b), e.map(x)) = left_open_interval(a, b, x)
} by {
    order_iso_map_is_order_embedding(e)
    is_order_embedding(e.map)
    order_embedding_left_open_interval_iff(e.map, a, b, x)
    left_open_interval(e.map(a), e.map(b), e.map(x)) = left_open_interval(a, b, x)
}

/// An order isomorphism sends right-open interval membership to right-open interval membership.
theorem order_iso_right_open_interval_image[A: LinearOrder, B: LinearOrder](e: OrderIso[A, B], a: A, b: A, x: A) {
    right_open_interval(a, b, x) implies right_open_interval(e.map(a), e.map(b), e.map(x))
} by {
    if right_open_interval(a, b, x) {
        order_iso_map_is_order_embedding(e)
        is_order_embedding(e.map)
        order_embedding_right_open_interval_image(e.map, a, b, x)
        right_open_interval(e.map(a), e.map(b), e.map(x))
    }
}

/// An order isomorphism reflects right-open interval membership.
theorem order_iso_right_open_interval_preimage[A: LinearOrder, B: LinearOrder](e: OrderIso[A, B], a: A, b: A, x: A) {
    right_open_interval(e.map(a), e.map(b), e.map(x)) implies right_open_interval(a, b, x)
} by {
    if right_open_interval(e.map(a), e.map(b), e.map(x)) {
        order_iso_map_is_order_embedding(e)
        is_order_embedding(e.map)
        order_embedding_right_open_interval_preimage(e.map, a, b, x)
        right_open_interval(a, b, x)
    }
}

/// An order isomorphism preserves and reflects right-open interval membership.
theorem order_iso_right_open_interval_iff[A: LinearOrder, B: LinearOrder](e: OrderIso[A, B], a: A, b: A, x: A) {
    right_open_interval(e.map(a), e.map(b), e.map(x)) = right_open_interval(a, b, x)
} by {
    order_iso_map_is_order_embedding(e)
    is_order_embedding(e.map)
    order_embedding_right_open_interval_iff(e.map, a, b, x)
    right_open_interval(e.map(a), e.map(b), e.map(x)) = right_open_interval(a, b, x)
}

/// An order isomorphism preserves binary minima.
theorem order_iso_min_image[A: LinearOrder, B: LinearOrder](e: OrderIso[A, B], a: A, b: A) {
    e.map(a.min(b)) = e.map(a).min(e.map(b))
} by {
    order_iso_map_is_order_embedding(e)
    order_embedding_min_image(e.map, a, b)
    e.map(a.min(b)) = e.map(a).min(e.map(b))
}

/// An order isomorphism preserves binary maxima.
theorem order_iso_max_image[A: LinearOrder, B: LinearOrder](e: OrderIso[A, B], a: A, b: A) {
    e.map(a.max(b)) = e.map(a).max(e.map(b))
} by {
    order_iso_map_is_order_embedding(e)
    order_embedding_max_image(e.map, a, b)
    e.map(a.max(b)) = e.map(a).max(e.map(b))
}

/// The inverse of an order isomorphism preserves binary minima.
theorem order_iso_inv_min_image[A: LinearOrder, B: LinearOrder](e: OrderIso[A, B], a: B, b: B) {
    e.inv(a.min(b)) = e.inv(a).min(e.inv(b))
} by {
    order_iso_inv_is_order_embedding(e)
    order_embedding_min_image(e.inv, a, b)
    e.inv(a.min(b)) = e.inv(a).min(e.inv(b))
}

/// The inverse of an order isomorphism preserves binary maxima.
theorem order_iso_inv_max_image[A: LinearOrder, B: LinearOrder](e: OrderIso[A, B], a: B, b: B) {
    e.inv(a.max(b)) = e.inv(a).max(e.inv(b))
} by {
    order_iso_inv_is_order_embedding(e)
    order_embedding_max_image(e.inv, a, b)
    e.inv(a.max(b)) = e.inv(a).max(e.inv(b))
}

/// An order-dual isomorphism sends closed-interval membership to reversed closed-interval membership.
theorem order_dual_iso_closed_interval_image[A: LinearOrder, B: LinearOrder](
    e: OrderDualIso[A, B],
    a: A,
    b: A,
    x: A
) {
    closed_interval(a, b, x) implies closed_interval(e.map(b), e.map(a), e.map(x))
} by {
    if closed_interval(a, b, x) {
        order_dual_iso_map_is_antitone(e)
        is_antitone(e.map)
        antitone_closed_interval_image(e.map, a, b, x)
        closed_interval(e.map(b), e.map(a), e.map(x))
    }
}

/// An order-dual isomorphism reflects reversed closed-interval membership.
theorem order_dual_iso_closed_interval_preimage[A: LinearOrder, B: LinearOrder](
    e: OrderDualIso[A, B],
    a: A,
    b: A,
    x: A
) {
    closed_interval(e.map(b), e.map(a), e.map(x)) implies closed_interval(a, b, x)
} by {
    if closed_interval(e.map(b), e.map(a), e.map(x)) {
        order_dual_iso_inv_is_antitone(e)
        is_antitone(e.inv)
        antitone_closed_interval_image(e.inv, e.map(b), e.map(a), e.map(x))
        closed_interval(e.inv(e.map(a)), e.inv(e.map(b)), e.inv(e.map(x)))
        order_dual_iso_left_inverse(e, a)
        order_dual_iso_left_inverse(e, b)
        order_dual_iso_left_inverse(e, x)
        closed_interval(a, b, x)
    }
}

/// An order-dual isomorphism preserves and reflects closed intervals with reversed endpoints.
theorem order_dual_iso_closed_interval_iff[A: LinearOrder, B: LinearOrder](
    e: OrderDualIso[A, B],
    a: A,
    b: A,
    x: A
) {
    closed_interval(e.map(b), e.map(a), e.map(x)) = closed_interval(a, b, x)
} by {
    if closed_interval(e.map(b), e.map(a), e.map(x)) {
        order_dual_iso_closed_interval_preimage(e, a, b, x)
        closed_interval(a, b, x)
    }
    if closed_interval(a, b, x) {
        order_dual_iso_closed_interval_image(e, a, b, x)
        closed_interval(e.map(b), e.map(a), e.map(x))
    }
    closed_interval(e.map(b), e.map(a), e.map(x)) = closed_interval(a, b, x)
}

/// An order-dual isomorphism sends open-interval membership to reversed open-interval membership.
theorem order_dual_iso_open_interval_image[A: LinearOrder, B: LinearOrder](
    e: OrderDualIso[A, B],
    a: A,
    b: A,
    x: A
) {
    open_interval(a, b, x) implies open_interval(e.map(b), e.map(a), e.map(x))
} by {
    if open_interval(a, b, x) {
        order_dual_iso_map_is_strict_antitone(e)
        is_strict_antitone(e.map)
        strict_antitone_open_interval_image(e.map, a, b, x)
        open_interval(e.map(b), e.map(a), e.map(x))
    }
}

/// An order-dual isomorphism reflects reversed open-interval membership.
theorem order_dual_iso_open_interval_preimage[A: LinearOrder, B: LinearOrder](
    e: OrderDualIso[A, B],
    a: A,
    b: A,
    x: A
) {
    open_interval(e.map(b), e.map(a), e.map(x)) implies open_interval(a, b, x)
} by {
    if open_interval(e.map(b), e.map(a), e.map(x)) {
        order_dual_iso_inv_is_strict_antitone(e)
        is_strict_antitone(e.inv)
        strict_antitone_open_interval_image(e.inv, e.map(b), e.map(a), e.map(x))
        open_interval(e.inv(e.map(a)), e.inv(e.map(b)), e.inv(e.map(x)))
        order_dual_iso_left_inverse(e, a)
        order_dual_iso_left_inverse(e, b)
        order_dual_iso_left_inverse(e, x)
        open_interval(a, b, x)
    }
}

/// An order-dual isomorphism preserves and reflects open intervals with reversed endpoints.
theorem order_dual_iso_open_interval_iff[A: LinearOrder, B: LinearOrder](
    e: OrderDualIso[A, B],
    a: A,
    b: A,
    x: A
) {
    open_interval(e.map(b), e.map(a), e.map(x)) = open_interval(a, b, x)
} by {
    if open_interval(e.map(b), e.map(a), e.map(x)) {
        order_dual_iso_open_interval_preimage(e, a, b, x)
        open_interval(a, b, x)
    }
    if open_interval(a, b, x) {
        order_dual_iso_open_interval_image(e, a, b, x)
        open_interval(e.map(b), e.map(a), e.map(x))
    }
    open_interval(e.map(b), e.map(a), e.map(x)) = open_interval(a, b, x)
}

/// An order-dual isomorphism sends left-open interval membership to reversed right-open interval membership.
theorem order_dual_iso_left_open_interval_image[A: LinearOrder, B: LinearOrder](
    e: OrderDualIso[A, B],
    a: A,
    b: A,
    x: A
) {
    left_open_interval(a, b, x) implies right_open_interval(e.map(b), e.map(a), e.map(x))
} by {
    if left_open_interval(a, b, x) {
        order_dual_iso_map_is_antitone(e)
        is_antitone(e.map)
        order_dual_iso_map_is_strict_antitone(e)
        is_strict_antitone(e.map)
        antitone_left_open_interval_image(e.map, a, b, x)
        right_open_interval(e.map(b), e.map(a), e.map(x))
    }
}

/// An order-dual isomorphism reflects reversed right-open interval membership to left-open membership.
theorem order_dual_iso_left_open_interval_preimage[A: LinearOrder, B: LinearOrder](
    e: OrderDualIso[A, B],
    a: A,
    b: A,
    x: A
) {
    right_open_interval(e.map(b), e.map(a), e.map(x)) implies left_open_interval(a, b, x)
} by {
    if right_open_interval(e.map(b), e.map(a), e.map(x)) {
        order_dual_iso_inv_is_antitone(e)
        is_antitone(e.inv)
        order_dual_iso_inv_is_strict_antitone(e)
        is_strict_antitone(e.inv)
        antitone_right_open_interval_image(e.inv, e.map(b), e.map(a), e.map(x))
        left_open_interval(e.inv(e.map(a)), e.inv(e.map(b)), e.inv(e.map(x)))
        order_dual_iso_left_inverse(e, a)
        order_dual_iso_left_inverse(e, b)
        order_dual_iso_left_inverse(e, x)
        left_open_interval(a, b, x)
    }
}

/// An order-dual isomorphism identifies left-open intervals with reversed right-open intervals.
theorem order_dual_iso_left_open_interval_iff[A: LinearOrder, B: LinearOrder](
    e: OrderDualIso[A, B],
    a: A,
    b: A,
    x: A
) {
    right_open_interval(e.map(b), e.map(a), e.map(x)) = left_open_interval(a, b, x)
} by {
    if right_open_interval(e.map(b), e.map(a), e.map(x)) {
        order_dual_iso_left_open_interval_preimage(e, a, b, x)
        left_open_interval(a, b, x)
    }
    if left_open_interval(a, b, x) {
        order_dual_iso_left_open_interval_image(e, a, b, x)
        right_open_interval(e.map(b), e.map(a), e.map(x))
    }
    right_open_interval(e.map(b), e.map(a), e.map(x)) = left_open_interval(a, b, x)
}

/// An order-dual isomorphism sends right-open interval membership to reversed left-open interval membership.
theorem order_dual_iso_right_open_interval_image[A: LinearOrder, B: LinearOrder](
    e: OrderDualIso[A, B],
    a: A,
    b: A,
    x: A
) {
    right_open_interval(a, b, x) implies left_open_interval(e.map(b), e.map(a), e.map(x))
} by {
    if right_open_interval(a, b, x) {
        order_dual_iso_map_is_antitone(e)
        is_antitone(e.map)
        order_dual_iso_map_is_strict_antitone(e)
        is_strict_antitone(e.map)
        antitone_right_open_interval_image(e.map, a, b, x)
        left_open_interval(e.map(b), e.map(a), e.map(x))
    }
}

/// An order-dual isomorphism reflects reversed left-open interval membership to right-open membership.
theorem order_dual_iso_right_open_interval_preimage[A: LinearOrder, B: LinearOrder](
    e: OrderDualIso[A, B],
    a: A,
    b: A,
    x: A
) {
    left_open_interval(e.map(b), e.map(a), e.map(x)) implies right_open_interval(a, b, x)
} by {
    if left_open_interval(e.map(b), e.map(a), e.map(x)) {
        order_dual_iso_inv_is_antitone(e)
        is_antitone(e.inv)
        order_dual_iso_inv_is_strict_antitone(e)
        is_strict_antitone(e.inv)
        antitone_left_open_interval_image(e.inv, e.map(b), e.map(a), e.map(x))
        right_open_interval(e.inv(e.map(a)), e.inv(e.map(b)), e.inv(e.map(x)))
        order_dual_iso_left_inverse(e, a)
        order_dual_iso_left_inverse(e, b)
        order_dual_iso_left_inverse(e, x)
        right_open_interval(a, b, x)
    }
}

/// An order-dual isomorphism identifies right-open intervals with reversed left-open intervals.
theorem order_dual_iso_right_open_interval_iff[A: LinearOrder, B: LinearOrder](
    e: OrderDualIso[A, B],
    a: A,
    b: A,
    x: A
) {
    left_open_interval(e.map(b), e.map(a), e.map(x)) = right_open_interval(a, b, x)
} by {
    if left_open_interval(e.map(b), e.map(a), e.map(x)) {
        order_dual_iso_right_open_interval_preimage(e, a, b, x)
        right_open_interval(a, b, x)
    }
    if right_open_interval(a, b, x) {
        order_dual_iso_right_open_interval_image(e, a, b, x)
        left_open_interval(e.map(b), e.map(a), e.map(x))
    }
    left_open_interval(e.map(b), e.map(a), e.map(x)) = right_open_interval(a, b, x)
}

/// An order-dual isomorphism sends binary minima to binary maxima.
theorem order_dual_iso_min_image[A: LinearOrder, B: LinearOrder](e: OrderDualIso[A, B], a: A, b: A) {
    e.map(a.min(b)) = e.map(a).max(e.map(b))
} by {
    if a <= b {
        min_eq_left_of_lte(a, b)
        a.min(b) = a
        order_dual_iso_apply_le(e, a, b)
        e.map(b) <= e.map(a)
        e.map(a) >= e.map(b)
        max_eq_left_of_gte(e.map(a), e.map(b))
        e.map(a).max(e.map(b)) = e.map(a)
        e.map(a.min(b)) = e.map(a).max(e.map(b))
    } else {
        b <= a
        a >= b
        min_eq_right_of_gte(a, b)
        a.min(b) = b
        order_dual_iso_apply_le(e, b, a)
        e.map(a) <= e.map(b)
        max_eq_right_of_lte(e.map(a), e.map(b))
        e.map(a).max(e.map(b)) = e.map(b)
        e.map(a.min(b)) = e.map(a).max(e.map(b))
    }
}

/// An order-dual isomorphism sends binary maxima to binary minima.
theorem order_dual_iso_max_image[A: LinearOrder, B: LinearOrder](e: OrderDualIso[A, B], a: A, b: A) {
    e.map(a.max(b)) = e.map(a).min(e.map(b))
} by {
    if a <= b {
        max_eq_right_of_lte(a, b)
        a.max(b) = b
        order_dual_iso_apply_le(e, a, b)
        e.map(b) <= e.map(a)
        e.map(a) >= e.map(b)
        min_eq_right_of_gte(e.map(a), e.map(b))
        e.map(a).min(e.map(b)) = e.map(b)
        e.map(a.max(b)) = e.map(a).min(e.map(b))
    } else {
        b <= a
        a >= b
        max_eq_left_of_gte(a, b)
        a.max(b) = a
        order_dual_iso_apply_le(e, b, a)
        e.map(a) <= e.map(b)
        min_eq_left_of_lte(e.map(a), e.map(b))
        e.map(a).min(e.map(b)) = e.map(a)
        e.map(a.max(b)) = e.map(a).min(e.map(b))
    }
}

/// The inverse of an order-dual isomorphism sends binary minima to binary maxima.
theorem order_dual_iso_inv_min_image[A: LinearOrder, B: LinearOrder](e: OrderDualIso[A, B], a: B, b: B) {
    e.inv(a.min(b)) = e.inv(a).max(e.inv(b))
} by {
    order_dual_iso_min_image(order_dual_iso_inverse(e), a, b)
    order_dual_iso_inverse_map(e)
    order_dual_iso_inverse(e).map = e.inv
    e.inv(a.min(b)) = e.inv(a).max(e.inv(b))
}

/// The inverse of an order-dual isomorphism sends binary maxima to binary minima.
theorem order_dual_iso_inv_max_image[A: LinearOrder, B: LinearOrder](e: OrderDualIso[A, B], a: B, b: B) {
    e.inv(a.max(b)) = e.inv(a).min(e.inv(b))
} by {
    order_dual_iso_max_image(order_dual_iso_inverse(e), a, b)
    order_dual_iso_inverse_map(e)
    order_dual_iso_inverse(e).map = e.inv
    e.inv(a.max(b)) = e.inv(a).min(e.inv(b))
}

/// An order isomorphism preserves binary infima.
theorem order_iso_meet_image[A: MeetSemilattice, B: MeetSemilattice](e: OrderIso[A, B], a: A, b: A) {
    e.map(a.meet(b)) = e.map(a).meet(e.map(b))
} by {
    meet_lte_left(a, b)
    order_iso_apply_le(e, a.meet(b), a)
    e.map(a.meet(b)) <= e.map(a)
    meet_lte_right(a, b)
    order_iso_apply_le(e, a.meet(b), b)
    e.map(a.meet(b)) <= e.map(b)
    lte_meet_of_bounds(e.map(a.meet(b)), e.map(a), e.map(b))
    e.map(a.meet(b)) <= e.map(a).meet(e.map(b))

    meet_lte_left(e.map(a), e.map(b))
    order_iso_inv_apply_le(e, e.map(a).meet(e.map(b)), e.map(a))
    e.inv(e.map(a).meet(e.map(b))) <= e.inv(e.map(a))
    order_iso_left_inverse(e, a)
    e.inv(e.map(a).meet(e.map(b))) <= a

    meet_lte_right(e.map(a), e.map(b))
    order_iso_inv_apply_le(e, e.map(a).meet(e.map(b)), e.map(b))
    e.inv(e.map(a).meet(e.map(b))) <= e.inv(e.map(b))
    order_iso_left_inverse(e, b)
    e.inv(e.map(a).meet(e.map(b))) <= b

    lte_meet_of_bounds(e.inv(e.map(a).meet(e.map(b))), a, b)
    e.inv(e.map(a).meet(e.map(b))) <= a.meet(b)
    order_iso_apply_le(e, e.inv(e.map(a).meet(e.map(b))), a.meet(b))
    e.map(e.inv(e.map(a).meet(e.map(b)))) <= e.map(a.meet(b))
    order_iso_right_inverse(e, e.map(a).meet(e.map(b)))
    e.map(a).meet(e.map(b)) <= e.map(a.meet(b))
}

/// An order isomorphism preserves binary suprema.
theorem order_iso_join_image[A: JoinSemilattice, B: JoinSemilattice](e: OrderIso[A, B], a: A, b: A) {
    e.map(a.join(b)) = e.map(a).join(e.map(b))
} by {
    lte_join_left(a, b)
    order_iso_apply_le(e, a, a.join(b))
    e.map(a) <= e.map(a.join(b))
    lte_join_right(a, b)
    order_iso_apply_le(e, b, a.join(b))
    e.map(b) <= e.map(a.join(b))
    join_lte_of_bounds(e.map(a), e.map(b), e.map(a.join(b)))
    e.map(a).join(e.map(b)) <= e.map(a.join(b))

    lte_join_left(e.map(a), e.map(b))
    order_iso_inv_apply_le(e, e.map(a), e.map(a).join(e.map(b)))
    e.inv(e.map(a)) <= e.inv(e.map(a).join(e.map(b)))
    order_iso_left_inverse(e, a)
    a <= e.inv(e.map(a).join(e.map(b)))

    lte_join_right(e.map(a), e.map(b))
    order_iso_inv_apply_le(e, e.map(b), e.map(a).join(e.map(b)))
    e.inv(e.map(b)) <= e.inv(e.map(a).join(e.map(b)))
    order_iso_left_inverse(e, b)
    b <= e.inv(e.map(a).join(e.map(b)))

    join_lte_of_bounds(a, b, e.inv(e.map(a).join(e.map(b))))
    a.join(b) <= e.inv(e.map(a).join(e.map(b)))
    order_iso_apply_le(e, a.join(b), e.inv(e.map(a).join(e.map(b))))
    e.map(a.join(b)) <= e.map(e.inv(e.map(a).join(e.map(b))))
    order_iso_right_inverse(e, e.map(a).join(e.map(b)))
    e.map(a.join(b)) <= e.map(a).join(e.map(b))
}

/// An order isomorphism preserves binary infima.
theorem order_iso_inf_image[A: MeetSemilattice, B: MeetSemilattice](e: OrderIso[A, B], a: A, b: A) {
    e.map(a.meet(b)) = e.map(a).meet(e.map(b))
} by {
    order_iso_meet_image(e, a, b)
}

/// An order isomorphism preserves binary suprema.
theorem order_iso_sup_image[A: JoinSemilattice, B: JoinSemilattice](e: OrderIso[A, B], a: A, b: A) {
    e.map(a.join(b)) = e.map(a).join(e.map(b))
} by {
    order_iso_join_image(e, a, b)
}

/// The inverse of an order isomorphism preserves binary infima.
theorem order_iso_inv_meet_image[A: MeetSemilattice, B: MeetSemilattice](e: OrderIso[A, B], a: B, b: B) {
    e.inv(a.meet(b)) = e.inv(a).meet(e.inv(b))
} by {
    meet_lte_left(a, b)
    order_iso_inv_apply_le(e, a.meet(b), a)
    e.inv(a.meet(b)) <= e.inv(a)
    meet_lte_right(a, b)
    order_iso_inv_apply_le(e, a.meet(b), b)
    e.inv(a.meet(b)) <= e.inv(b)
    lte_meet_of_bounds(e.inv(a.meet(b)), e.inv(a), e.inv(b))
    e.inv(a.meet(b)) <= e.inv(a).meet(e.inv(b))

    meet_lte_left(e.inv(a), e.inv(b))
    order_iso_apply_le(e, e.inv(a).meet(e.inv(b)), e.inv(a))
    e.map(e.inv(a).meet(e.inv(b))) <= e.map(e.inv(a))
    order_iso_right_inverse(e, a)
    e.map(e.inv(a).meet(e.inv(b))) <= a

    meet_lte_right(e.inv(a), e.inv(b))
    order_iso_apply_le(e, e.inv(a).meet(e.inv(b)), e.inv(b))
    e.map(e.inv(a).meet(e.inv(b))) <= e.map(e.inv(b))
    order_iso_right_inverse(e, b)
    e.map(e.inv(a).meet(e.inv(b))) <= b

    lte_meet_of_bounds(e.map(e.inv(a).meet(e.inv(b))), a, b)
    e.map(e.inv(a).meet(e.inv(b))) <= a.meet(b)
    order_iso_inv_apply_le(e, e.map(e.inv(a).meet(e.inv(b))), a.meet(b))
    e.inv(e.map(e.inv(a).meet(e.inv(b)))) <= e.inv(a.meet(b))
    order_iso_left_inverse(e, e.inv(a).meet(e.inv(b)))
    e.inv(a).meet(e.inv(b)) <= e.inv(a.meet(b))
}

/// The inverse of an order isomorphism preserves binary suprema.
theorem order_iso_inv_join_image[A: JoinSemilattice, B: JoinSemilattice](e: OrderIso[A, B], a: B, b: B) {
    e.inv(a.join(b)) = e.inv(a).join(e.inv(b))
} by {
    lte_join_left(a, b)
    order_iso_inv_apply_le(e, a, a.join(b))
    e.inv(a) <= e.inv(a.join(b))
    lte_join_right(a, b)
    order_iso_inv_apply_le(e, b, a.join(b))
    e.inv(b) <= e.inv(a.join(b))
    join_lte_of_bounds(e.inv(a), e.inv(b), e.inv(a.join(b)))
    e.inv(a).join(e.inv(b)) <= e.inv(a.join(b))

    lte_join_left(e.inv(a), e.inv(b))
    order_iso_apply_le(e, e.inv(a), e.inv(a).join(e.inv(b)))
    e.map(e.inv(a)) <= e.map(e.inv(a).join(e.inv(b)))
    order_iso_right_inverse(e, a)
    a <= e.map(e.inv(a).join(e.inv(b)))

    lte_join_right(e.inv(a), e.inv(b))
    order_iso_apply_le(e, e.inv(b), e.inv(a).join(e.inv(b)))
    e.map(e.inv(b)) <= e.map(e.inv(a).join(e.inv(b)))
    order_iso_right_inverse(e, b)
    b <= e.map(e.inv(a).join(e.inv(b)))

    join_lte_of_bounds(a, b, e.map(e.inv(a).join(e.inv(b))))
    a.join(b) <= e.map(e.inv(a).join(e.inv(b)))
    order_iso_inv_apply_le(e, a.join(b), e.map(e.inv(a).join(e.inv(b))))
    e.inv(a.join(b)) <= e.inv(e.map(e.inv(a).join(e.inv(b))))
    order_iso_left_inverse(e, e.inv(a).join(e.inv(b)))
    e.inv(a.join(b)) <= e.inv(a).join(e.inv(b))
}

/// The inverse of an order isomorphism preserves binary infima.
theorem order_iso_inv_inf_image[A: MeetSemilattice, B: MeetSemilattice](e: OrderIso[A, B], a: B, b: B) {
    e.inv(a.meet(b)) = e.inv(a).meet(e.inv(b))
} by {
    order_iso_inv_meet_image(e, a, b)
}

/// The inverse of an order isomorphism preserves binary suprema.
theorem order_iso_inv_sup_image[A: JoinSemilattice, B: JoinSemilattice](e: OrderIso[A, B], a: B, b: B) {
    e.inv(a.join(b)) = e.inv(a).join(e.inv(b))
} by {
    order_iso_inv_join_image(e, a, b)
}

/// An order-dual isomorphism carries binary infima to binary suprema.
theorem order_dual_iso_meet_image[A: MeetSemilattice, B: JoinSemilattice](e: OrderDualIso[A, B], a: A, b: A) {
    e.map(a.meet(b)) = e.map(a).join(e.map(b))
} by {
    meet_lte_left(a, b)
    order_dual_iso_apply_le(e, a.meet(b), a)
    e.map(a) <= e.map(a.meet(b))
    meet_lte_right(a, b)
    order_dual_iso_apply_le(e, a.meet(b), b)
    e.map(b) <= e.map(a.meet(b))
    join_lte_of_bounds(e.map(a), e.map(b), e.map(a.meet(b)))
    e.map(a).join(e.map(b)) <= e.map(a.meet(b))

    order_dual_iso_inv_is_antitone(e)
    lte_join_left(e.map(a), e.map(b))
    antitone_apply(e.inv, e.map(a), e.map(a).join(e.map(b)))
    e.inv(e.map(a).join(e.map(b))) <= e.inv(e.map(a))
    order_dual_iso_left_inverse(e, a)
    e.inv(e.map(a).join(e.map(b))) <= a

    lte_join_right(e.map(a), e.map(b))
    antitone_apply(e.inv, e.map(b), e.map(a).join(e.map(b)))
    e.inv(e.map(a).join(e.map(b))) <= e.inv(e.map(b))
    order_dual_iso_left_inverse(e, b)
    e.inv(e.map(a).join(e.map(b))) <= b

    lte_meet_of_bounds(e.inv(e.map(a).join(e.map(b))), a, b)
    e.inv(e.map(a).join(e.map(b))) <= a.meet(b)
    order_dual_iso_apply_le(e, e.inv(e.map(a).join(e.map(b))), a.meet(b))
    e.map(a.meet(b)) <= e.map(e.inv(e.map(a).join(e.map(b))))
    order_dual_iso_right_inverse(e, e.map(a).join(e.map(b)))
    e.map(a.meet(b)) <= e.map(a).join(e.map(b))
}

/// An order-dual isomorphism carries binary suprema to binary infima.
theorem order_dual_iso_join_image[A: JoinSemilattice, B: MeetSemilattice](e: OrderDualIso[A, B], a: A, b: A) {
    e.map(a.join(b)) = e.map(a).meet(e.map(b))
} by {
    lte_join_left(a, b)
    order_dual_iso_apply_le(e, a, a.join(b))
    e.map(a.join(b)) <= e.map(a)
    lte_join_right(a, b)
    order_dual_iso_apply_le(e, b, a.join(b))
    e.map(a.join(b)) <= e.map(b)
    lte_meet_of_bounds(e.map(a.join(b)), e.map(a), e.map(b))
    e.map(a.join(b)) <= e.map(a).meet(e.map(b))

    order_dual_iso_inv_is_antitone(e)
    meet_lte_left(e.map(a), e.map(b))
    antitone_apply(e.inv, e.map(a).meet(e.map(b)), e.map(a))
    e.inv(e.map(a)) <= e.inv(e.map(a).meet(e.map(b)))
    order_dual_iso_left_inverse(e, a)
    a <= e.inv(e.map(a).meet(e.map(b)))

    meet_lte_right(e.map(a), e.map(b))
    antitone_apply(e.inv, e.map(a).meet(e.map(b)), e.map(b))
    e.inv(e.map(b)) <= e.inv(e.map(a).meet(e.map(b)))
    order_dual_iso_left_inverse(e, b)
    b <= e.inv(e.map(a).meet(e.map(b)))

    join_lte_of_bounds(a, b, e.inv(e.map(a).meet(e.map(b))))
    a.join(b) <= e.inv(e.map(a).meet(e.map(b)))
    order_dual_iso_apply_le(e, a.join(b), e.inv(e.map(a).meet(e.map(b))))
    e.map(e.inv(e.map(a).meet(e.map(b)))) <= e.map(a.join(b))
    order_dual_iso_right_inverse(e, e.map(a).meet(e.map(b)))
    e.map(a).meet(e.map(b)) <= e.map(a.join(b))
}

/// The inverse of an order-dual isomorphism carries binary infima to binary suprema.
theorem order_dual_iso_inv_meet_image[A: JoinSemilattice, B: MeetSemilattice](e: OrderDualIso[A, B], a: B, b: B) {
    e.inv(a.meet(b)) = e.inv(a).join(e.inv(b))
} by {
    order_dual_iso_inverse_map(e)
    order_dual_iso_inverse_inv(e)
    order_dual_iso_meet_image(order_dual_iso_inverse(e), a, b)
}

/// The inverse of an order-dual isomorphism carries binary suprema to binary infima.
theorem order_dual_iso_inv_join_image[A: MeetSemilattice, B: JoinSemilattice](e: OrderDualIso[A, B], a: B, b: B) {
    e.inv(a.join(b)) = e.inv(a).meet(e.inv(b))
} by {
    order_dual_iso_inverse_map(e)
    order_dual_iso_inverse_inv(e)
    order_dual_iso_join_image(order_dual_iso_inverse(e), a, b)
}

/// The forward function associated to an order isomorphism.
define order_iso_apply[A: PartialOrder, B: PartialOrder](e: OrderIso[A, B], x: A) -> B {
    e.map(x)
}

/// The inverse function associated to an order isomorphism.
define order_iso_unapply[A: PartialOrder, B: PartialOrder](e: OrderIso[A, B], y: B) -> A {
    e.inv(y)
}

/// Applying an order isomorphism is applying its underlying map.
theorem order_iso_apply_at[A: PartialOrder, B: PartialOrder](e: OrderIso[A, B], x: A) {
    order_iso_apply(e, x) = e.map(x)
}

/// Unapplying an order isomorphism is applying its inverse map.
theorem order_iso_unapply_at[A: PartialOrder, B: PartialOrder](e: OrderIso[A, B], y: B) {
    order_iso_unapply(e, y) = e.inv(y)
}

/// Unapplying after applying an order isomorphism gives the original point.
theorem order_iso_unapply_apply[A: PartialOrder, B: PartialOrder](e: OrderIso[A, B], x: A) {
    order_iso_unapply(e, order_iso_apply(e, x)) = x
} by {
    order_iso_apply_at(e, x)
    order_iso_unapply_at(e, order_iso_apply(e, x))
    order_iso_left_inverse(e, x)
}

/// Applying after unapplying an order isomorphism gives the original point.
theorem order_iso_apply_unapply[A: PartialOrder, B: PartialOrder](e: OrderIso[A, B], y: B) {
    order_iso_apply(e, order_iso_unapply(e, y)) = y
} by {
    order_iso_unapply_at(e, y)
    order_iso_apply_at(e, order_iso_unapply(e, y))
    order_iso_right_inverse(e, y)
}

/// The wrapper function of an order isomorphism is injective.
theorem order_iso_apply_is_injective[A: PartialOrder, B: PartialOrder](e: OrderIso[A, B]) {
    is_injective_fn(order_iso_apply(e))
} by {
    forall(x: A, y: A) {
        if order_iso_apply(e, x) = order_iso_apply(e, y) {
            order_iso_unapply_apply(e, x)
            order_iso_unapply_apply(e, y)
            order_iso_unapply(e, order_iso_apply(e, x)) =
                order_iso_unapply(e, order_iso_apply(e, y))
            x = y
        }
    }
}

/// The wrapper function of an order isomorphism is surjective.
theorem order_iso_apply_is_surjective[A: PartialOrder, B: PartialOrder](e: OrderIso[A, B]) {
    is_surjective_fn(order_iso_apply(e))
} by {
    forall(y: B) {
        exists(x: A) {
            x = order_iso_unapply(e, y) and order_iso_apply(e, x) = y
        }
    }
}

/// The wrapper function of an order isomorphism is bijective.
theorem order_iso_apply_is_bijection[A: PartialOrder, B: PartialOrder](e: OrderIso[A, B]) {
    is_bijection_fn(order_iso_apply(e))
} by {
    order_iso_apply_is_injective(e)
    order_iso_apply_is_surjective(e)
}

/// The inverse wrapper function of an order isomorphism is injective.
theorem order_iso_unapply_is_injective[A: PartialOrder, B: PartialOrder](e: OrderIso[A, B]) {
    is_injective_fn(order_iso_unapply(e))
} by {
    forall(x: B, y: B) {
        if order_iso_unapply(e, x) = order_iso_unapply(e, y) {
            order_iso_apply_unapply(e, x)
            order_iso_apply_unapply(e, y)
            order_iso_apply(e, order_iso_unapply(e, x)) =
                order_iso_apply(e, order_iso_unapply(e, y))
            x = y
        }
    }
}

/// The inverse wrapper function of an order isomorphism is surjective.
theorem order_iso_unapply_is_surjective[A: PartialOrder, B: PartialOrder](e: OrderIso[A, B]) {
    is_surjective_fn(order_iso_unapply(e))
} by {
    forall(x: A) {
        exists(y: B) {
            y = order_iso_apply(e, x) and order_iso_unapply(e, y) = x
        }
    }
}

/// The inverse wrapper function of an order isomorphism is bijective.
theorem order_iso_unapply_is_bijection[A: PartialOrder, B: PartialOrder](e: OrderIso[A, B]) {
    is_bijection_fn(order_iso_unapply(e))
} by {
    order_iso_unapply_is_injective(e)
    order_iso_unapply_is_surjective(e)
}

/// The wrapper function of an order isomorphism is an order embedding.
theorem order_iso_apply_is_order_embedding[A: PartialOrder, B: PartialOrder](e: OrderIso[A, B]) {
    is_order_embedding(order_iso_apply(e))
} by {
    forall(x: A, y: A) {
        if order_iso_apply(e, x) <= order_iso_apply(e, y) {
            order_iso_apply_at(e, x)
            order_iso_apply_at(e, y)
            e.map(x) <= e.map(y)
            order_iso_reflects_le(e, x, y)
            x <= y
        }
        if x <= y {
            order_iso_apply_le(e, x, y)
            e.map(x) <= e.map(y)
            order_iso_apply_at(e, x)
            order_iso_apply_at(e, y)
            order_iso_apply(e, x) <= order_iso_apply(e, y)
        }
        order_iso_apply(e, x) <= order_iso_apply(e, y) = (x <= y)
    }
}

/// The inverse wrapper function of an order isomorphism is an order embedding.
theorem order_iso_unapply_is_order_embedding[A: PartialOrder, B: PartialOrder](e: OrderIso[A, B]) {
    is_order_embedding(order_iso_unapply(e))
} by {
    forall(x: B, y: B) {
        if order_iso_unapply(e, x) <= order_iso_unapply(e, y) {
            order_iso_unapply_at(e, x)
            order_iso_unapply_at(e, y)
            e.inv(x) <= e.inv(y)
            order_iso_inv_reflects_le(e, x, y)
            x <= y
        }
        if x <= y {
            order_iso_inv_apply_le(e, x, y)
            e.inv(x) <= e.inv(y)
            order_iso_unapply_at(e, x)
            order_iso_unapply_at(e, y)
            order_iso_unapply(e, x) <= order_iso_unapply(e, y)
        }
        order_iso_unapply(e, x) <= order_iso_unapply(e, y) = (x <= y)
    }
}

/// The order-isomorphism wrapper preserves non-strict order.
theorem order_iso_apply_fn_le[A: PartialOrder, B: PartialOrder](e: OrderIso[A, B], x: A, y: A) {
    x <= y implies order_iso_apply(e, x) <= order_iso_apply(e, y)
} by {
    if x <= y {
        order_iso_apply_le(e, x, y)
        order_iso_apply_at(e, x)
        order_iso_apply_at(e, y)
        order_iso_apply(e, x) <= order_iso_apply(e, y)
    }
}

/// The order-isomorphism wrapper reflects non-strict order.
theorem order_iso_apply_fn_reflects_le[A: PartialOrder, B: PartialOrder](e: OrderIso[A, B], x: A, y: A) {
    order_iso_apply(e, x) <= order_iso_apply(e, y) implies x <= y
} by {
    if order_iso_apply(e, x) <= order_iso_apply(e, y) {
        order_iso_apply_at(e, x)
        order_iso_apply_at(e, y)
        order_iso_reflects_le(e, x, y)
        x <= y
    }
}

/// The order-isomorphism wrapper preserves and reflects non-strict order.
theorem order_iso_apply_fn_le_iff[A: PartialOrder, B: PartialOrder](e: OrderIso[A, B], x: A, y: A) {
    order_iso_apply(e, x) <= order_iso_apply(e, y) = (x <= y)
} by {
    order_iso_apply_is_order_embedding(e)
    order_embedding_le_iff_le(order_iso_apply(e), x, y)
}

/// The order-isomorphism wrapper preserves strict order.
theorem order_iso_apply_fn_lt[A: PartialOrder, B: PartialOrder](e: OrderIso[A, B], x: A, y: A) {
    x < y implies order_iso_apply(e, x) < order_iso_apply(e, y)
} by {
    if x < y {
        order_iso_apply_lt(e, x, y)
        order_iso_apply_at(e, x)
        order_iso_apply_at(e, y)
        order_iso_apply(e, x) < order_iso_apply(e, y)
    }
}

/// The order-isomorphism wrapper reflects strict order.
theorem order_iso_apply_fn_reflects_lt[A: PartialOrder, B: PartialOrder](e: OrderIso[A, B], x: A, y: A) {
    order_iso_apply(e, x) < order_iso_apply(e, y) implies x < y
} by {
    if order_iso_apply(e, x) < order_iso_apply(e, y) {
        order_iso_apply_at(e, x)
        order_iso_apply_at(e, y)
        order_iso_reflects_lt(e, x, y)
        x < y
    }
}

/// The order-isomorphism wrapper preserves and reflects strict order.
theorem order_iso_apply_fn_lt_iff[A: PartialOrder, B: PartialOrder](e: OrderIso[A, B], x: A, y: A) {
    order_iso_apply(e, x) < order_iso_apply(e, y) = (x < y)
} by {
    if order_iso_apply(e, x) < order_iso_apply(e, y) {
        order_iso_apply_fn_reflects_lt(e, x, y)
        x < y
    }
    if x < y {
        order_iso_apply_fn_lt(e, x, y)
        order_iso_apply(e, x) < order_iso_apply(e, y)
    }
    order_iso_apply(e, x) < order_iso_apply(e, y) = (x < y)
}

/// The order-isomorphism wrapper preserves reverse non-strict order.
theorem order_iso_apply_fn_ge[A: PartialOrder, B: PartialOrder](e: OrderIso[A, B], x: A, y: A) {
    x >= y implies order_iso_apply(e, x) >= order_iso_apply(e, y)
} by {
    if x >= y {
        order_iso_apply_ge(e, x, y)
        order_iso_apply_at(e, x)
        order_iso_apply_at(e, y)
        order_iso_apply(e, x) >= order_iso_apply(e, y)
    }
}

/// The order-isomorphism wrapper reflects reverse non-strict order.
theorem order_iso_apply_fn_reflects_ge[A: PartialOrder, B: PartialOrder](e: OrderIso[A, B], x: A, y: A) {
    order_iso_apply(e, x) >= order_iso_apply(e, y) implies x >= y
} by {
    if order_iso_apply(e, x) >= order_iso_apply(e, y) {
        order_iso_apply_at(e, x)
        order_iso_apply_at(e, y)
        order_iso_reflects_ge(e, x, y)
        x >= y
    }
}

/// The order-isomorphism wrapper preserves and reflects reverse non-strict order.
theorem order_iso_apply_fn_ge_iff[A: PartialOrder, B: PartialOrder](e: OrderIso[A, B], x: A, y: A) {
    order_iso_apply(e, x) >= order_iso_apply(e, y) = (x >= y)
} by {
    if order_iso_apply(e, x) >= order_iso_apply(e, y) {
        order_iso_apply_fn_reflects_ge(e, x, y)
        x >= y
    }
    if x >= y {
        order_iso_apply_fn_ge(e, x, y)
        order_iso_apply(e, x) >= order_iso_apply(e, y)
    }
    order_iso_apply(e, x) >= order_iso_apply(e, y) = (x >= y)
}

/// The order-isomorphism wrapper preserves strict reverse order.
theorem order_iso_apply_fn_gt[A: PartialOrder, B: PartialOrder](e: OrderIso[A, B], x: A, y: A) {
    x > y implies order_iso_apply(e, x) > order_iso_apply(e, y)
} by {
    if x > y {
        order_iso_apply_gt(e, x, y)
        order_iso_apply_at(e, x)
        order_iso_apply_at(e, y)
        order_iso_apply(e, x) > order_iso_apply(e, y)
    }
}

/// The order-isomorphism wrapper reflects strict reverse order.
theorem order_iso_apply_fn_reflects_gt[A: PartialOrder, B: PartialOrder](e: OrderIso[A, B], x: A, y: A) {
    order_iso_apply(e, x) > order_iso_apply(e, y) implies x > y
} by {
    if order_iso_apply(e, x) > order_iso_apply(e, y) {
        order_iso_apply_at(e, x)
        order_iso_apply_at(e, y)
        order_iso_reflects_gt(e, x, y)
        x > y
    }
}

/// The order-isomorphism wrapper preserves and reflects strict reverse order.
theorem order_iso_apply_fn_gt_iff[A: PartialOrder, B: PartialOrder](e: OrderIso[A, B], x: A, y: A) {
    order_iso_apply(e, x) > order_iso_apply(e, y) = (x > y)
} by {
    if order_iso_apply(e, x) > order_iso_apply(e, y) {
        order_iso_apply_fn_reflects_gt(e, x, y)
        x > y
    }
    if x > y {
        order_iso_apply_fn_gt(e, x, y)
        order_iso_apply(e, x) > order_iso_apply(e, y)
    }
    order_iso_apply(e, x) > order_iso_apply(e, y) = (x > y)
}

/// The inverse order-isomorphism wrapper preserves non-strict order.
theorem order_iso_unapply_fn_le[A: PartialOrder, B: PartialOrder](e: OrderIso[A, B], x: B, y: B) {
    x <= y implies order_iso_unapply(e, x) <= order_iso_unapply(e, y)
} by {
    if x <= y {
        order_iso_inv_apply_le(e, x, y)
        order_iso_unapply_at(e, x)
        order_iso_unapply_at(e, y)
        order_iso_unapply(e, x) <= order_iso_unapply(e, y)
    }
}

/// The inverse order-isomorphism wrapper reflects non-strict order.
theorem order_iso_unapply_fn_reflects_le[A: PartialOrder, B: PartialOrder](e: OrderIso[A, B], x: B, y: B) {
    order_iso_unapply(e, x) <= order_iso_unapply(e, y) implies x <= y
} by {
    if order_iso_unapply(e, x) <= order_iso_unapply(e, y) {
        order_iso_unapply_at(e, x)
        order_iso_unapply_at(e, y)
        order_iso_inv_reflects_le(e, x, y)
        x <= y
    }
}

/// The inverse order-isomorphism wrapper preserves and reflects non-strict order.
theorem order_iso_unapply_fn_le_iff[A: PartialOrder, B: PartialOrder](e: OrderIso[A, B], x: B, y: B) {
    order_iso_unapply(e, x) <= order_iso_unapply(e, y) = (x <= y)
} by {
    order_iso_unapply_is_order_embedding(e)
    order_embedding_le_iff_le(order_iso_unapply(e), x, y)
}

/// The inverse order-isomorphism wrapper preserves strict order.
theorem order_iso_unapply_fn_lt[A: PartialOrder, B: PartialOrder](e: OrderIso[A, B], x: B, y: B) {
    x < y implies order_iso_unapply(e, x) < order_iso_unapply(e, y)
} by {
    if x < y {
        order_iso_inv_apply_lt(e, x, y)
        order_iso_unapply_at(e, x)
        order_iso_unapply_at(e, y)
        order_iso_unapply(e, x) < order_iso_unapply(e, y)
    }
}

/// The inverse order-isomorphism wrapper reflects strict order.
theorem order_iso_unapply_fn_reflects_lt[A: PartialOrder, B: PartialOrder](e: OrderIso[A, B], x: B, y: B) {
    order_iso_unapply(e, x) < order_iso_unapply(e, y) implies x < y
} by {
    if order_iso_unapply(e, x) < order_iso_unapply(e, y) {
        order_iso_unapply_at(e, x)
        order_iso_unapply_at(e, y)
        order_iso_inv_reflects_lt(e, x, y)
        x < y
    }
}

/// The inverse order-isomorphism wrapper preserves and reflects strict order.
theorem order_iso_unapply_fn_lt_iff[A: PartialOrder, B: PartialOrder](e: OrderIso[A, B], x: B, y: B) {
    order_iso_unapply(e, x) < order_iso_unapply(e, y) = (x < y)
} by {
    if order_iso_unapply(e, x) < order_iso_unapply(e, y) {
        order_iso_unapply_fn_reflects_lt(e, x, y)
        x < y
    }
    if x < y {
        order_iso_unapply_fn_lt(e, x, y)
        order_iso_unapply(e, x) < order_iso_unapply(e, y)
    }
    order_iso_unapply(e, x) < order_iso_unapply(e, y) = (x < y)
}

/// The inverse order-isomorphism wrapper preserves reverse non-strict order.
theorem order_iso_unapply_fn_ge[A: PartialOrder, B: PartialOrder](e: OrderIso[A, B], x: B, y: B) {
    x >= y implies order_iso_unapply(e, x) >= order_iso_unapply(e, y)
} by {
    if x >= y {
        order_iso_inv_apply_ge(e, x, y)
        order_iso_unapply_at(e, x)
        order_iso_unapply_at(e, y)
        order_iso_unapply(e, x) >= order_iso_unapply(e, y)
    }
}

/// The inverse order-isomorphism wrapper reflects reverse non-strict order.
theorem order_iso_unapply_fn_reflects_ge[A: PartialOrder, B: PartialOrder](e: OrderIso[A, B], x: B, y: B) {
    order_iso_unapply(e, x) >= order_iso_unapply(e, y) implies x >= y
} by {
    if order_iso_unapply(e, x) >= order_iso_unapply(e, y) {
        order_iso_unapply_at(e, x)
        order_iso_unapply_at(e, y)
        order_iso_inv_reflects_ge(e, x, y)
        x >= y
    }
}

/// The inverse order-isomorphism wrapper preserves and reflects reverse non-strict order.
theorem order_iso_unapply_fn_ge_iff[A: PartialOrder, B: PartialOrder](e: OrderIso[A, B], x: B, y: B) {
    order_iso_unapply(e, x) >= order_iso_unapply(e, y) = (x >= y)
} by {
    if order_iso_unapply(e, x) >= order_iso_unapply(e, y) {
        order_iso_unapply_fn_reflects_ge(e, x, y)
        x >= y
    }
    if x >= y {
        order_iso_unapply_fn_ge(e, x, y)
        order_iso_unapply(e, x) >= order_iso_unapply(e, y)
    }
    order_iso_unapply(e, x) >= order_iso_unapply(e, y) = (x >= y)
}

/// The inverse order-isomorphism wrapper preserves strict reverse order.
theorem order_iso_unapply_fn_gt[A: PartialOrder, B: PartialOrder](e: OrderIso[A, B], x: B, y: B) {
    x > y implies order_iso_unapply(e, x) > order_iso_unapply(e, y)
} by {
    if x > y {
        order_iso_inv_apply_gt(e, x, y)
        order_iso_unapply_at(e, x)
        order_iso_unapply_at(e, y)
        order_iso_unapply(e, x) > order_iso_unapply(e, y)
    }
}

/// The inverse order-isomorphism wrapper reflects strict reverse order.
theorem order_iso_unapply_fn_reflects_gt[A: PartialOrder, B: PartialOrder](e: OrderIso[A, B], x: B, y: B) {
    order_iso_unapply(e, x) > order_iso_unapply(e, y) implies x > y
} by {
    if order_iso_unapply(e, x) > order_iso_unapply(e, y) {
        order_iso_unapply_at(e, x)
        order_iso_unapply_at(e, y)
        order_iso_inv_reflects_gt(e, x, y)
        x > y
    }
}

/// The inverse order-isomorphism wrapper preserves and reflects strict reverse order.
theorem order_iso_unapply_fn_gt_iff[A: PartialOrder, B: PartialOrder](e: OrderIso[A, B], x: B, y: B) {
    order_iso_unapply(e, x) > order_iso_unapply(e, y) = (x > y)
} by {
    if order_iso_unapply(e, x) > order_iso_unapply(e, y) {
        order_iso_unapply_fn_reflects_gt(e, x, y)
        x > y
    }
    if x > y {
        order_iso_unapply_fn_gt(e, x, y)
        order_iso_unapply(e, x) > order_iso_unapply(e, y)
    }
    order_iso_unapply(e, x) > order_iso_unapply(e, y) = (x > y)
}

/// The identity order isomorphism wrapper is the identity function at a point.
theorem identity_order_iso_apply[A: PartialOrder](anchor: A, x: A) {
    order_iso_apply(identity_order_iso(anchor), x) = x
} by {
    identity_order_iso_map(anchor)
    identity_order_iso(anchor).map = identity_fn[A]
    order_iso_apply(identity_order_iso(anchor), x) = identity_order_iso(anchor).map(x)
    identity_fn[A](x) = x
}

/// The identity order isomorphism inverse wrapper is the identity function at a point.
theorem identity_order_iso_unapply[A: PartialOrder](anchor: A, x: A) {
    order_iso_unapply(identity_order_iso(anchor), x) = x
} by {
    identity_order_iso_inv(anchor)
    identity_order_iso(anchor).inv = identity_fn[A]
    order_iso_unapply(identity_order_iso(anchor), x) = identity_order_iso(anchor).inv(x)
    identity_fn[A](x) = x
}

/// The inverse order isomorphism wrapper applies the original inverse map.
theorem inverse_order_iso_apply[A: PartialOrder, B: PartialOrder](e: OrderIso[A, B], y: B) {
    order_iso_apply(inverse_order_iso(e), y) = order_iso_unapply(e, y)
} by {
    inverse_order_iso_map(e)
    inverse_order_iso(e).map = e.inv
    order_iso_apply(inverse_order_iso(e), y) = inverse_order_iso(e).map(y)
    order_iso_unapply(e, y) = e.inv(y)
}

/// The inverse order isomorphism inverse wrapper applies the original map.
theorem inverse_order_iso_unapply[A: PartialOrder, B: PartialOrder](e: OrderIso[A, B], x: A) {
    order_iso_unapply(inverse_order_iso(e), x) = order_iso_apply(e, x)
} by {
    inverse_order_iso_inv(e)
    inverse_order_iso(e).inv = e.map
    order_iso_unapply(inverse_order_iso(e), x) = inverse_order_iso(e).inv(x)
    order_iso_apply(e, x) = e.map(x)
}

/// Applying a composite order isomorphism is composing the wrapper functions.
theorem compose_order_iso_apply[A: PartialOrder, B: PartialOrder, C: PartialOrder](
    e: OrderIso[B, C],
    h: OrderIso[A, B],
    x: A
) {
    order_iso_apply(compose_order_iso(e, h), x) =
    order_iso_apply(e, order_iso_apply(h, x))
} by {
    compose_order_iso_map(e, h)
    compose_order_iso(e, h).map = compose(e.map, h.map)
    order_iso_apply(compose_order_iso(e, h), x) = compose_order_iso(e, h).map(x)
    compose(e.map, h.map, x) = e.map(h.map(x))
    order_iso_apply(h, x) = h.map(x)
    order_iso_apply(e, order_iso_apply(h, x)) = e.map(order_iso_apply(h, x))
    order_iso_apply(e, order_iso_apply(h, x)) = e.map(h.map(x))
}

/// Unapplying a composite order isomorphism is composing the inverse wrapper functions.
theorem compose_order_iso_unapply[A: PartialOrder, B: PartialOrder, C: PartialOrder](
    e: OrderIso[B, C],
    h: OrderIso[A, B],
    z: C
) {
    order_iso_unapply(compose_order_iso(e, h), z) =
    order_iso_unapply(h, order_iso_unapply(e, z))
} by {
    compose_order_iso_inv(e, h)
    compose_order_iso(e, h).inv = compose(h.inv, e.inv)
    order_iso_unapply(compose_order_iso(e, h), z) = compose_order_iso(e, h).inv(z)
    compose(h.inv, e.inv, z) = h.inv(e.inv(z))
    order_iso_unapply(e, z) = e.inv(z)
    order_iso_unapply(h, order_iso_unapply(e, z)) = h.inv(order_iso_unapply(e, z))
    order_iso_unapply(h, order_iso_unapply(e, z)) = h.inv(e.inv(z))
}

/// True if one function lies pointwise below another.
define function_order_lte[X, A: PartialOrder](f: X -> A, g: X -> A) -> Bool {
    forall(x: X) {
        f(x) <= g(x)
    }
}

/// True if one function lies pointwise strictly below another.
define function_order_lt[X, A: PartialOrder](f: X -> A, g: X -> A) -> Bool {
    forall(x: X) {
        f(x) < g(x)
    }
}

/// The pointwise order on functions is reflexive.
theorem function_order_lte_refl[X, A: PartialOrder](f: X -> A) {
    function_order_lte(f, f)
} by {
    function_order_lte(f, f) = forall(x: X) {
        f(x) <= f(x)
    }
}

/// A pointwise order comparison gives the comparison at each point.
theorem function_order_lte_at[X, A: PartialOrder](f: X -> A, g: X -> A, x: X) {
    function_order_lte(f, g) implies f(x) <= g(x)
} by {
    if function_order_lte(f, g) {
        function_order_lte(f, g) = forall(y: X) {
            f(y) <= g(y)
        }
        f(x) <= g(x)
    }
}

/// The pointwise order on functions is transitive.
theorem function_order_lte_trans[X, A: PartialOrder](f: X -> A, g: X -> A, h: X -> A) {
    function_order_lte(f, g) and function_order_lte(g, h) implies function_order_lte(f, h)
} by {
    if function_order_lte(f, g) and function_order_lte(g, h) {
        function_order_lte(f, g) = forall(x: X) {
            f(x) <= g(x)
        }
        function_order_lte(g, h) = forall(x: X) {
            g(x) <= h(x)
        }
        forall(x: X) {
            f(x) <= g(x)
            g(x) <= h(x)
            f(x) <= h(x)
        }
        function_order_lte(f, h) = forall(x: X) {
            f(x) <= h(x)
        }
        function_order_lte(f, h)
    }
}

/// Pointwise strict order implies pointwise non-strict order.
theorem function_order_lt_imp_lte[X, A: PartialOrder](f: X -> A, g: X -> A) {
    function_order_lt(f, g) implies function_order_lte(f, g)
} by {
    if function_order_lt(f, g) {
        function_order_lt(f, g) = forall(x: X) {
            f(x) < g(x)
        }
        forall(x: X) {
            f(x) < g(x)
            f(x) <= g(x)
        }
        function_order_lte(f, g) = forall(x: X) {
            f(x) <= g(x)
        }
        function_order_lte(f, g)
    }
}

/// A pointwise strict comparison gives the strict comparison at each point.
theorem function_order_lt_at[X, A: PartialOrder](f: X -> A, g: X -> A, x: X) {
    function_order_lt(f, g) implies f(x) < g(x)
} by {
    if function_order_lt(f, g) {
        function_order_lt(f, g) = forall(y: X) {
            f(y) < g(y)
        }
        f(x) < g(x)
    }
}

/// The pointwise strict order is transitive.
theorem function_order_lt_trans[X, A: PartialOrder](f: X -> A, g: X -> A, h: X -> A) {
    function_order_lt(f, g) and function_order_lt(g, h) implies function_order_lt(f, h)
} by {
    if function_order_lt(f, g) and function_order_lt(g, h) {
        function_order_lt(f, g) = forall(x: X) {
            f(x) < g(x)
        }
        function_order_lt(g, h) = forall(x: X) {
            g(x) < h(x)
        }
        forall(x: X) {
            f(x) < g(x)
            g(x) < h(x)
            lt_trans(f(x), g(x), h(x))
            f(x) < h(x)
        }
        function_order_lt(f, h) = forall(x: X) {
            f(x) < h(x)
        }
        function_order_lt(f, h)
    }
}

/// Applying an order isomorphism pointwise to a function.
define function_order_iso_map[X, A: PartialOrder, B: PartialOrder](
    e: OrderIso[A, B],
    f: X -> A,
    x: X
) -> B {
    e.map(f(x))
}

/// Applying the inverse of an order isomorphism pointwise to a function.
define function_order_iso_unmap[X, A: PartialOrder, B: PartialOrder](
    e: OrderIso[A, B],
    g: X -> B,
    x: X
) -> A {
    e.inv(g(x))
}

/// Pointwise application of an order isomorphism has the inverse pointwise application as a left inverse.
theorem function_order_iso_unmap_map[X, A: PartialOrder, B: PartialOrder](
    e: OrderIso[A, B],
    f: X -> A
) {
    function_order_iso_unmap(e, function_order_iso_map(e, f)) = f
} by {
    forall(x: X) {
        function_order_iso_map(e, f, x) = e.map(f(x))
        function_order_iso_unmap(e, function_order_iso_map(e, f), x) =
            e.inv(function_order_iso_map(e, f, x))
        e.inv(function_order_iso_map(e, f, x)) = e.inv(e.map(f(x)))
        order_iso_left_inverse(e, f(x))
        e.inv(e.map(f(x))) = f(x)
        function_order_iso_unmap(e, function_order_iso_map(e, f), x) = f(x)
    }
    function_extensionality(function_order_iso_unmap(e, function_order_iso_map(e, f)), f)
}

/// Pointwise application of an order isomorphism has the inverse pointwise application as a right inverse.
theorem function_order_iso_map_unmap[X, A: PartialOrder, B: PartialOrder](
    e: OrderIso[A, B],
    g: X -> B
) {
    function_order_iso_map(e, function_order_iso_unmap(e, g)) = g
} by {
    forall(x: X) {
        function_order_iso_unmap(e, g, x) = e.inv(g(x))
        function_order_iso_map(e, function_order_iso_unmap(e, g), x) =
            e.map(function_order_iso_unmap(e, g, x))
        e.map(function_order_iso_unmap(e, g, x)) = e.map(e.inv(g(x)))
        order_iso_right_inverse(e, g(x))
        e.map(e.inv(g(x))) = g(x)
        function_order_iso_map(e, function_order_iso_unmap(e, g), x) = g(x)
    }
    function_extensionality(function_order_iso_map(e, function_order_iso_unmap(e, g)), g)
}

/// Pointwise application of the identity order isomorphism is the original function.
theorem function_order_iso_map_identity[X, A: PartialOrder](anchor: A, f: X -> A) {
    function_order_iso_map(identity_order_iso(anchor), f) = f
} by {
    forall(x: X) {
        identity_order_iso_map(anchor)
        identity_order_iso(anchor).map = identity_fn[A]
        function_order_iso_map(identity_order_iso(anchor), f, x) =
            identity_order_iso(anchor).map(f(x))
        identity_fn[A](f(x)) = f(x)
        function_order_iso_map(identity_order_iso(anchor), f, x) = f(x)
    }
    function_extensionality(function_order_iso_map(identity_order_iso(anchor), f), f)
}

/// Pointwise inverse application of the identity order isomorphism is the original function.
theorem function_order_iso_unmap_identity[X, A: PartialOrder](anchor: A, f: X -> A) {
    function_order_iso_unmap(identity_order_iso(anchor), f) = f
} by {
    forall(x: X) {
        identity_order_iso_inv(anchor)
        identity_order_iso(anchor).inv = identity_fn[A]
        function_order_iso_unmap(identity_order_iso(anchor), f, x) =
            identity_order_iso(anchor).inv(f(x))
        identity_fn[A](f(x)) = f(x)
        function_order_iso_unmap(identity_order_iso(anchor), f, x) = f(x)
    }
    function_extensionality(function_order_iso_unmap(identity_order_iso(anchor), f), f)
}

/// Pointwise application of an inverse order isomorphism is pointwise inverse application.
theorem function_order_iso_map_inverse[X, A: PartialOrder, B: PartialOrder](
    e: OrderIso[A, B],
    g: X -> B
) {
    function_order_iso_map(inverse_order_iso(e), g) = function_order_iso_unmap(e, g)
} by {
    forall(x: X) {
        inverse_order_iso_map(e)
        inverse_order_iso(e).map = e.inv
        function_order_iso_map(inverse_order_iso(e), g, x) =
            inverse_order_iso(e).map(g(x))
        function_order_iso_unmap(e, g, x) = e.inv(g(x))
        function_order_iso_map(inverse_order_iso(e), g, x) = function_order_iso_unmap(e, g, x)
    }
    function_extensionality(function_order_iso_map(inverse_order_iso(e), g), function_order_iso_unmap(e, g))
}

/// Pointwise inverse application of an inverse order isomorphism is pointwise application.
theorem function_order_iso_unmap_inverse[X, A: PartialOrder, B: PartialOrder](
    e: OrderIso[A, B],
    f: X -> A
) {
    function_order_iso_unmap(inverse_order_iso(e), f) = function_order_iso_map(e, f)
} by {
    forall(x: X) {
        inverse_order_iso_inv(e)
        inverse_order_iso(e).inv = e.map
        function_order_iso_unmap(inverse_order_iso(e), f, x) =
            inverse_order_iso(e).inv(f(x))
        function_order_iso_map(e, f, x) = e.map(f(x))
        function_order_iso_unmap(inverse_order_iso(e), f, x) = function_order_iso_map(e, f, x)
    }
    function_extensionality(function_order_iso_unmap(inverse_order_iso(e), f), function_order_iso_map(e, f))
}

/// Pointwise application of a composite order isomorphism is composite pointwise application.
theorem function_order_iso_map_compose[X, A: PartialOrder, B: PartialOrder, C: PartialOrder](
    e: OrderIso[B, C],
    h: OrderIso[A, B],
    f: X -> A
) {
    function_order_iso_map(compose_order_iso(e, h), f) =
    function_order_iso_map(e, function_order_iso_map(h, f))
} by {
    forall(x: X) {
        compose_order_iso_map(e, h)
        compose_order_iso(e, h).map = compose(e.map, h.map)
        function_order_iso_map(compose_order_iso(e, h), f, x) =
            compose_order_iso(e, h).map(f(x))
        compose(e.map, h.map, f(x)) = e.map(h.map(f(x)))
        function_order_iso_map(h, f, x) = h.map(f(x))
        function_order_iso_map(e, function_order_iso_map(h, f), x) =
            e.map(function_order_iso_map(h, f, x))
        function_order_iso_map(e, function_order_iso_map(h, f), x) = e.map(h.map(f(x)))
        function_order_iso_map(compose_order_iso(e, h), f, x) =
            function_order_iso_map(e, function_order_iso_map(h, f), x)
    }
    function_extensionality(
        function_order_iso_map(compose_order_iso(e, h), f),
        function_order_iso_map(e, function_order_iso_map(h, f))
    )
}

/// Pointwise inverse application of a composite order isomorphism is composite pointwise inverse application.
theorem function_order_iso_unmap_compose[X, A: PartialOrder, B: PartialOrder, C: PartialOrder](
    e: OrderIso[B, C],
    h: OrderIso[A, B],
    f: X -> C
) {
    function_order_iso_unmap(compose_order_iso(e, h), f) =
    function_order_iso_unmap(h, function_order_iso_unmap(e, f))
} by {
    forall(x: X) {
        compose_order_iso_inv(e, h)
        compose_order_iso(e, h).inv = compose(h.inv, e.inv)
        function_order_iso_unmap(compose_order_iso(e, h), f, x) =
            compose_order_iso(e, h).inv(f(x))
        compose(h.inv, e.inv, f(x)) = h.inv(e.inv(f(x)))
        function_order_iso_unmap(e, f, x) = e.inv(f(x))
        function_order_iso_unmap(h, function_order_iso_unmap(e, f), x) =
            h.inv(function_order_iso_unmap(e, f, x))
        function_order_iso_unmap(h, function_order_iso_unmap(e, f), x) = h.inv(e.inv(f(x)))
        function_order_iso_unmap(compose_order_iso(e, h), f, x) =
            function_order_iso_unmap(h, function_order_iso_unmap(e, f), x)
    }
    function_extensionality(
        function_order_iso_unmap(compose_order_iso(e, h), f),
        function_order_iso_unmap(h, function_order_iso_unmap(e, f))
    )
}

/// Pointwise application of an order isomorphism preserves the pointwise order.
theorem function_order_iso_map_lte[X, A: PartialOrder, B: PartialOrder](
    e: OrderIso[A, B],
    f: X -> A,
    g: X -> A
) {
    function_order_lte(f, g) implies
    function_order_lte(function_order_iso_map(e, f), function_order_iso_map(e, g))
} by {
    if function_order_lte(f, g) {
        function_order_lte(f, g) = forall(x: X) {
            f(x) <= g(x)
        }
        forall(x: X) {
            f(x) <= g(x)
            order_iso_apply_le(e, f(x), g(x))
            e.map(f(x)) <= e.map(g(x))
            function_order_iso_map(e, f, x) = e.map(f(x))
            function_order_iso_map(e, g, x) = e.map(g(x))
            function_order_iso_map(e, f, x) <= function_order_iso_map(e, g, x)
        }
        function_order_lte(function_order_iso_map(e, f), function_order_iso_map(e, g)) =
            forall(x: X) {
                function_order_iso_map(e, f, x) <= function_order_iso_map(e, g, x)
            }
        function_order_lte(function_order_iso_map(e, f), function_order_iso_map(e, g))
    }
}

/// Pointwise application of an order isomorphism reflects the pointwise order.
theorem function_order_iso_map_reflects_lte[X, A: PartialOrder, B: PartialOrder](
    e: OrderIso[A, B],
    f: X -> A,
    g: X -> A
) {
    function_order_lte(function_order_iso_map(e, f), function_order_iso_map(e, g))
    implies
    function_order_lte(f, g)
} by {
    if function_order_lte(function_order_iso_map(e, f), function_order_iso_map(e, g)) {
        function_order_lte(function_order_iso_map(e, f), function_order_iso_map(e, g)) =
            forall(x: X) {
                function_order_iso_map(e, f, x) <= function_order_iso_map(e, g, x)
            }
        forall(x: X) {
            function_order_iso_map(e, f, x) <= function_order_iso_map(e, g, x)
            function_order_iso_map(e, f, x) = e.map(f(x))
            function_order_iso_map(e, g, x) = e.map(g(x))
            e.map(f(x)) <= e.map(g(x))
            order_iso_reflects_le(e, f(x), g(x))
            f(x) <= g(x)
        }
        function_order_lte(f, g) = forall(x: X) {
            f(x) <= g(x)
        }
        function_order_lte(f, g)
    }
}

/// Pointwise application of an order isomorphism preserves and reflects the pointwise order.
theorem function_order_iso_map_lte_iff[X, A: PartialOrder, B: PartialOrder](
    e: OrderIso[A, B],
    f: X -> A,
    g: X -> A
) {
    function_order_lte(function_order_iso_map(e, f), function_order_iso_map(e, g)) =
    function_order_lte(f, g)
} by {
    if function_order_lte(function_order_iso_map(e, f), function_order_iso_map(e, g)) {
        function_order_iso_map_reflects_lte(e, f, g)
        function_order_lte(f, g)
    }
    if function_order_lte(f, g) {
        function_order_iso_map_lte(e, f, g)
        function_order_lte(function_order_iso_map(e, f), function_order_iso_map(e, g))
    }
    function_order_lte(function_order_iso_map(e, f), function_order_iso_map(e, g)) =
        function_order_lte(f, g)
}

/// Pointwise application of an order isomorphism preserves pointwise strict order.
theorem function_order_iso_map_lt[X, A: PartialOrder, B: PartialOrder](
    e: OrderIso[A, B],
    f: X -> A,
    g: X -> A
) {
    function_order_lt(f, g) implies
    function_order_lt(function_order_iso_map(e, f), function_order_iso_map(e, g))
} by {
    if function_order_lt(f, g) {
        function_order_lt(f, g) = forall(x: X) {
            f(x) < g(x)
        }
        forall(x: X) {
            f(x) < g(x)
            order_iso_apply_lt(e, f(x), g(x))
            e.map(f(x)) < e.map(g(x))
            function_order_iso_map(e, f, x) = e.map(f(x))
            function_order_iso_map(e, g, x) = e.map(g(x))
            function_order_iso_map(e, f, x) < function_order_iso_map(e, g, x)
        }
        function_order_lt(function_order_iso_map(e, f), function_order_iso_map(e, g)) =
            forall(x: X) {
                function_order_iso_map(e, f, x) < function_order_iso_map(e, g, x)
            }
        function_order_lt(function_order_iso_map(e, f), function_order_iso_map(e, g))
    }
}

/// Pointwise application of an order isomorphism reflects pointwise strict order.
theorem function_order_iso_map_reflects_lt[X, A: PartialOrder, B: PartialOrder](
    e: OrderIso[A, B],
    f: X -> A,
    g: X -> A
) {
    function_order_lt(function_order_iso_map(e, f), function_order_iso_map(e, g))
    implies
    function_order_lt(f, g)
} by {
    if function_order_lt(function_order_iso_map(e, f), function_order_iso_map(e, g)) {
        function_order_lt(function_order_iso_map(e, f), function_order_iso_map(e, g)) =
            forall(x: X) {
                function_order_iso_map(e, f, x) < function_order_iso_map(e, g, x)
            }
        forall(x: X) {
            function_order_iso_map(e, f, x) < function_order_iso_map(e, g, x)
            function_order_iso_map(e, f, x) = e.map(f(x))
            function_order_iso_map(e, g, x) = e.map(g(x))
            e.map(f(x)) < e.map(g(x))
            order_iso_reflects_lt(e, f(x), g(x))
            f(x) < g(x)
        }
        function_order_lt(f, g) = forall(x: X) {
            f(x) < g(x)
        }
        function_order_lt(f, g)
    }
}

/// Pointwise application of an order isomorphism preserves and reflects pointwise strict order.
theorem function_order_iso_map_lt_iff[X, A: PartialOrder, B: PartialOrder](
    e: OrderIso[A, B],
    f: X -> A,
    g: X -> A
) {
    function_order_lt(function_order_iso_map(e, f), function_order_iso_map(e, g)) =
    function_order_lt(f, g)
} by {
    if function_order_lt(function_order_iso_map(e, f), function_order_iso_map(e, g)) {
        function_order_iso_map_reflects_lt(e, f, g)
        function_order_lt(f, g)
    }
    if function_order_lt(f, g) {
        function_order_iso_map_lt(e, f, g)
        function_order_lt(function_order_iso_map(e, f), function_order_iso_map(e, g))
    }
    function_order_lt(function_order_iso_map(e, f), function_order_iso_map(e, g)) =
        function_order_lt(f, g)
}

/// Pointwise application of the inverse of an order isomorphism preserves the pointwise order.
theorem function_order_iso_unmap_lte[X, A: PartialOrder, B: PartialOrder](
    e: OrderIso[A, B],
    f: X -> B,
    g: X -> B
) {
    function_order_lte(f, g) implies
    function_order_lte(function_order_iso_unmap(e, f), function_order_iso_unmap(e, g))
} by {
    if function_order_lte(f, g) {
        function_order_lte(f, g) = forall(x: X) {
            f(x) <= g(x)
        }
        forall(x: X) {
            f(x) <= g(x)
            order_iso_inv_apply_le(e, f(x), g(x))
            e.inv(f(x)) <= e.inv(g(x))
            function_order_iso_unmap(e, f, x) = e.inv(f(x))
            function_order_iso_unmap(e, g, x) = e.inv(g(x))
            function_order_iso_unmap(e, f, x) <= function_order_iso_unmap(e, g, x)
        }
        function_order_lte(function_order_iso_unmap(e, f), function_order_iso_unmap(e, g)) =
            forall(x: X) {
                function_order_iso_unmap(e, f, x) <= function_order_iso_unmap(e, g, x)
            }
        function_order_lte(function_order_iso_unmap(e, f), function_order_iso_unmap(e, g))
    }
}

/// Pointwise application of the inverse of an order isomorphism reflects the pointwise order.
theorem function_order_iso_unmap_reflects_lte[X, A: PartialOrder, B: PartialOrder](
    e: OrderIso[A, B],
    f: X -> B,
    g: X -> B
) {
    function_order_lte(function_order_iso_unmap(e, f), function_order_iso_unmap(e, g))
    implies
    function_order_lte(f, g)
} by {
    if function_order_lte(function_order_iso_unmap(e, f), function_order_iso_unmap(e, g)) {
        function_order_lte(function_order_iso_unmap(e, f), function_order_iso_unmap(e, g)) =
            forall(x: X) {
                function_order_iso_unmap(e, f, x) <= function_order_iso_unmap(e, g, x)
            }
        forall(x: X) {
            function_order_iso_unmap(e, f, x) <= function_order_iso_unmap(e, g, x)
            function_order_iso_unmap(e, f, x) = e.inv(f(x))
            function_order_iso_unmap(e, g, x) = e.inv(g(x))
            e.inv(f(x)) <= e.inv(g(x))
            order_iso_inv_reflects_le(e, f(x), g(x))
            f(x) <= g(x)
        }
        function_order_lte(f, g) = forall(x: X) {
            f(x) <= g(x)
        }
        function_order_lte(f, g)
    }
}

/// Pointwise application of the inverse of an order isomorphism preserves and reflects the pointwise order.
theorem function_order_iso_unmap_lte_iff[X, A: PartialOrder, B: PartialOrder](
    e: OrderIso[A, B],
    f: X -> B,
    g: X -> B
) {
    function_order_lte(function_order_iso_unmap(e, f), function_order_iso_unmap(e, g)) =
    function_order_lte(f, g)
} by {
    if function_order_lte(function_order_iso_unmap(e, f), function_order_iso_unmap(e, g)) {
        function_order_iso_unmap_reflects_lte(e, f, g)
        function_order_lte(f, g)
    }
    if function_order_lte(f, g) {
        function_order_iso_unmap_lte(e, f, g)
        function_order_lte(function_order_iso_unmap(e, f), function_order_iso_unmap(e, g))
    }
    function_order_lte(function_order_iso_unmap(e, f), function_order_iso_unmap(e, g)) =
        function_order_lte(f, g)
}

/// Pointwise application of the inverse of an order isomorphism preserves pointwise strict order.
theorem function_order_iso_unmap_lt[X, A: PartialOrder, B: PartialOrder](
    e: OrderIso[A, B],
    f: X -> B,
    g: X -> B
) {
    function_order_lt(f, g) implies
    function_order_lt(function_order_iso_unmap(e, f), function_order_iso_unmap(e, g))
} by {
    if function_order_lt(f, g) {
        function_order_lt(f, g) = forall(x: X) {
            f(x) < g(x)
        }
        forall(x: X) {
            f(x) < g(x)
            order_iso_inv_apply_lt(e, f(x), g(x))
            e.inv(f(x)) < e.inv(g(x))
            function_order_iso_unmap(e, f, x) = e.inv(f(x))
            function_order_iso_unmap(e, g, x) = e.inv(g(x))
            function_order_iso_unmap(e, f, x) < function_order_iso_unmap(e, g, x)
        }
        function_order_lt(function_order_iso_unmap(e, f), function_order_iso_unmap(e, g)) =
            forall(x: X) {
                function_order_iso_unmap(e, f, x) < function_order_iso_unmap(e, g, x)
            }
        function_order_lt(function_order_iso_unmap(e, f), function_order_iso_unmap(e, g))
    }
}

/// Pointwise application of the inverse of an order isomorphism reflects pointwise strict order.
theorem function_order_iso_unmap_reflects_lt[X, A: PartialOrder, B: PartialOrder](
    e: OrderIso[A, B],
    f: X -> B,
    g: X -> B
) {
    function_order_lt(function_order_iso_unmap(e, f), function_order_iso_unmap(e, g))
    implies
    function_order_lt(f, g)
} by {
    if function_order_lt(function_order_iso_unmap(e, f), function_order_iso_unmap(e, g)) {
        function_order_lt(function_order_iso_unmap(e, f), function_order_iso_unmap(e, g)) =
            forall(x: X) {
                function_order_iso_unmap(e, f, x) < function_order_iso_unmap(e, g, x)
            }
        forall(x: X) {
            function_order_iso_unmap(e, f, x) < function_order_iso_unmap(e, g, x)
            function_order_iso_unmap(e, f, x) = e.inv(f(x))
            function_order_iso_unmap(e, g, x) = e.inv(g(x))
            e.inv(f(x)) < e.inv(g(x))
            order_iso_inv_reflects_lt(e, f(x), g(x))
            f(x) < g(x)
        }
        function_order_lt(f, g) = forall(x: X) {
            f(x) < g(x)
        }
        function_order_lt(f, g)
    }
}

/// Pointwise application of the inverse of an order isomorphism preserves and reflects pointwise strict order.
theorem function_order_iso_unmap_lt_iff[X, A: PartialOrder, B: PartialOrder](
    e: OrderIso[A, B],
    f: X -> B,
    g: X -> B
) {
    function_order_lt(function_order_iso_unmap(e, f), function_order_iso_unmap(e, g)) =
    function_order_lt(f, g)
} by {
    if function_order_lt(function_order_iso_unmap(e, f), function_order_iso_unmap(e, g)) {
        function_order_iso_unmap_reflects_lt(e, f, g)
        function_order_lt(f, g)
    }
    if function_order_lt(f, g) {
        function_order_iso_unmap_lt(e, f, g)
        function_order_lt(function_order_iso_unmap(e, f), function_order_iso_unmap(e, g))
    }
    function_order_lt(function_order_iso_unmap(e, f), function_order_iso_unmap(e, g)) =
        function_order_lt(f, g)
}

/// Reindexing a function by an order isomorphism of domains.
define function_order_iso_reindex[X: PartialOrder, Y: PartialOrder, A: PartialOrder](
    e: OrderIso[X, Y],
    f: Y -> A,
    x: X
) -> A {
    f(e.map(x))
}

/// Reindexing a function by the inverse order isomorphism of domains.
define function_order_iso_unreindex[X: PartialOrder, Y: PartialOrder, A: PartialOrder](
    e: OrderIso[X, Y],
    f: X -> A,
    y: Y
) -> A {
    f(e.inv(y))
}

/// Reindexing by an order isomorphism and then by its inverse gives the original function.
theorem function_order_iso_unreindex_reindex[X: PartialOrder, Y: PartialOrder, A: PartialOrder](
    e: OrderIso[X, Y],
    f: Y -> A
) {
    function_order_iso_unreindex(e, function_order_iso_reindex(e, f)) = f
} by {
    forall(y: Y) {
        function_order_iso_unreindex(e, function_order_iso_reindex(e, f), y) =
            function_order_iso_reindex(e, f, e.inv(y))
        function_order_iso_reindex(e, f, e.inv(y)) = f(e.map(e.inv(y)))
        order_iso_right_inverse(e, y)
        e.map(e.inv(y)) = y
        f(e.map(e.inv(y))) = f(y)
        function_order_iso_unreindex(e, function_order_iso_reindex(e, f), y) = f(y)
    }
    function_extensionality(function_order_iso_unreindex(e, function_order_iso_reindex(e, f)), f)
}

/// Reindexing by the inverse order isomorphism and then by the order isomorphism gives the original function.
theorem function_order_iso_reindex_unreindex[X: PartialOrder, Y: PartialOrder, A: PartialOrder](
    e: OrderIso[X, Y],
    f: X -> A
) {
    function_order_iso_reindex(e, function_order_iso_unreindex(e, f)) = f
} by {
    forall(x: X) {
        function_order_iso_reindex(e, function_order_iso_unreindex(e, f), x) =
            function_order_iso_unreindex(e, f, e.map(x))
        function_order_iso_unreindex(e, f, e.map(x)) = f(e.inv(e.map(x)))
        order_iso_left_inverse(e, x)
        e.inv(e.map(x)) = x
        f(e.inv(e.map(x))) = f(x)
        function_order_iso_reindex(e, function_order_iso_unreindex(e, f), x) = f(x)
    }
    function_extensionality(function_order_iso_reindex(e, function_order_iso_unreindex(e, f)), f)
}

/// Reindexing by the identity order isomorphism is the original function.
theorem function_order_iso_reindex_identity[X: PartialOrder, A: PartialOrder](anchor: X, f: X -> A) {
    function_order_iso_reindex(identity_order_iso(anchor), f) = f
} by {
    forall(x: X) {
        identity_order_iso_map(anchor)
        identity_order_iso(anchor).map = identity_fn[X]
        function_order_iso_reindex(identity_order_iso(anchor), f, x) =
            f(identity_order_iso(anchor).map(x))
        identity_fn[X](x) = x
        function_order_iso_reindex(identity_order_iso(anchor), f, x) = f(x)
    }
    function_extensionality(function_order_iso_reindex(identity_order_iso(anchor), f), f)
}

/// Unreindexing by the identity order isomorphism is the original function.
theorem function_order_iso_unreindex_identity[X: PartialOrder, A: PartialOrder](anchor: X, f: X -> A) {
    function_order_iso_unreindex(identity_order_iso(anchor), f) = f
} by {
    forall(x: X) {
        identity_order_iso_inv(anchor)
        identity_order_iso(anchor).inv = identity_fn[X]
        function_order_iso_unreindex(identity_order_iso(anchor), f, x) =
            f(identity_order_iso(anchor).inv(x))
        identity_fn[X](x) = x
        function_order_iso_unreindex(identity_order_iso(anchor), f, x) = f(x)
    }
    function_extensionality(function_order_iso_unreindex(identity_order_iso(anchor), f), f)
}

/// Reindexing by an inverse order isomorphism is unreindexing by the original isomorphism.
theorem function_order_iso_reindex_inverse[X: PartialOrder, Y: PartialOrder, A: PartialOrder](
    e: OrderIso[X, Y],
    f: X -> A
) {
    function_order_iso_reindex(inverse_order_iso(e), f) = function_order_iso_unreindex(e, f)
} by {
    forall(y: Y) {
        inverse_order_iso_map(e)
        inverse_order_iso(e).map = e.inv
        function_order_iso_reindex(inverse_order_iso(e), f, y) =
            f(inverse_order_iso(e).map(y))
        function_order_iso_unreindex(e, f, y) = f(e.inv(y))
        function_order_iso_reindex(inverse_order_iso(e), f, y) =
            function_order_iso_unreindex(e, f, y)
    }
    function_extensionality(function_order_iso_reindex(inverse_order_iso(e), f), function_order_iso_unreindex(e, f))
}

/// Unreindexing by an inverse order isomorphism is reindexing by the original isomorphism.
theorem function_order_iso_unreindex_inverse[X: PartialOrder, Y: PartialOrder, A: PartialOrder](
    e: OrderIso[X, Y],
    f: Y -> A
) {
    function_order_iso_unreindex(inverse_order_iso(e), f) = function_order_iso_reindex(e, f)
} by {
    forall(x: X) {
        inverse_order_iso_inv(e)
        inverse_order_iso(e).inv = e.map
        function_order_iso_unreindex(inverse_order_iso(e), f, x) =
            f(inverse_order_iso(e).inv(x))
        function_order_iso_reindex(e, f, x) = f(e.map(x))
        function_order_iso_unreindex(inverse_order_iso(e), f, x) =
            function_order_iso_reindex(e, f, x)
    }
    function_extensionality(function_order_iso_unreindex(inverse_order_iso(e), f), function_order_iso_reindex(e, f))
}

/// Reindexing by a composite order isomorphism is composite reindexing.
theorem function_order_iso_reindex_compose[X: PartialOrder, Y: PartialOrder, Z: PartialOrder, A: PartialOrder](
    e: OrderIso[Y, Z],
    h: OrderIso[X, Y],
    f: Z -> A
) {
    function_order_iso_reindex(compose_order_iso(e, h), f) =
    function_order_iso_reindex(h, function_order_iso_reindex(e, f))
} by {
    forall(x: X) {
        compose_order_iso_map(e, h)
        compose_order_iso(e, h).map = compose(e.map, h.map)
        function_order_iso_reindex(compose_order_iso(e, h), f, x) =
            f(compose_order_iso(e, h).map(x))
        compose(e.map, h.map, x) = e.map(h.map(x))
        function_order_iso_reindex(e, f, h.map(x)) = f(e.map(h.map(x)))
        function_order_iso_reindex(h, function_order_iso_reindex(e, f), x) =
            function_order_iso_reindex(e, f, h.map(x))
        function_order_iso_reindex(compose_order_iso(e, h), f, x) =
            function_order_iso_reindex(h, function_order_iso_reindex(e, f), x)
    }
    function_extensionality(
        function_order_iso_reindex(compose_order_iso(e, h), f),
        function_order_iso_reindex(h, function_order_iso_reindex(e, f))
    )
}

/// Unreindexing by a composite order isomorphism is composite unreindexing.
theorem function_order_iso_unreindex_compose[X: PartialOrder, Y: PartialOrder, Z: PartialOrder, A: PartialOrder](
    e: OrderIso[Y, Z],
    h: OrderIso[X, Y],
    f: X -> A
) {
    function_order_iso_unreindex(compose_order_iso(e, h), f) =
    function_order_iso_unreindex(e, function_order_iso_unreindex(h, f))
} by {
    forall(z: Z) {
        compose_order_iso_inv(e, h)
        compose_order_iso(e, h).inv = compose(h.inv, e.inv)
        function_order_iso_unreindex(compose_order_iso(e, h), f, z) =
            f(compose_order_iso(e, h).inv(z))
        compose(h.inv, e.inv, z) = h.inv(e.inv(z))
        function_order_iso_unreindex(h, f, e.inv(z)) = f(h.inv(e.inv(z)))
        function_order_iso_unreindex(e, function_order_iso_unreindex(h, f), z) =
            function_order_iso_unreindex(h, f, e.inv(z))
        function_order_iso_unreindex(compose_order_iso(e, h), f, z) =
            function_order_iso_unreindex(e, function_order_iso_unreindex(h, f), z)
    }
    function_extensionality(
        function_order_iso_unreindex(compose_order_iso(e, h), f),
        function_order_iso_unreindex(e, function_order_iso_unreindex(h, f))
    )
}

/// Reindexing by an order isomorphism preserves pointwise order.
theorem function_order_iso_reindex_lte[X: PartialOrder, Y: PartialOrder, A: PartialOrder](
    e: OrderIso[X, Y],
    f: Y -> A,
    g: Y -> A
) {
    function_order_lte(f, g) implies
    function_order_lte(function_order_iso_reindex(e, f), function_order_iso_reindex(e, g))
} by {
    if function_order_lte(f, g) {
        function_order_lte(f, g) = forall(y: Y) {
            f(y) <= g(y)
        }
        forall(x: X) {
            f(e.map(x)) <= g(e.map(x))
            function_order_iso_reindex(e, f, x) = f(e.map(x))
            function_order_iso_reindex(e, g, x) = g(e.map(x))
            function_order_iso_reindex(e, f, x) <= function_order_iso_reindex(e, g, x)
        }
        function_order_lte(function_order_iso_reindex(e, f), function_order_iso_reindex(e, g)) =
            forall(x: X) {
                function_order_iso_reindex(e, f, x) <= function_order_iso_reindex(e, g, x)
            }
        function_order_lte(function_order_iso_reindex(e, f), function_order_iso_reindex(e, g))
    }
}

/// Reindexing by an order isomorphism reflects pointwise order.
theorem function_order_iso_reindex_reflects_lte[X: PartialOrder, Y: PartialOrder, A: PartialOrder](
    e: OrderIso[X, Y],
    f: Y -> A,
    g: Y -> A
) {
    function_order_lte(function_order_iso_reindex(e, f), function_order_iso_reindex(e, g))
    implies
    function_order_lte(f, g)
} by {
    if function_order_lte(function_order_iso_reindex(e, f), function_order_iso_reindex(e, g)) {
        function_order_lte(function_order_iso_reindex(e, f), function_order_iso_reindex(e, g)) =
            forall(x: X) {
                function_order_iso_reindex(e, f, x) <= function_order_iso_reindex(e, g, x)
            }
        forall(y: Y) {
            function_order_iso_reindex(e, f, e.inv(y)) <= function_order_iso_reindex(e, g, e.inv(y))
            function_order_iso_reindex(e, f, e.inv(y)) = f(e.map(e.inv(y)))
            function_order_iso_reindex(e, g, e.inv(y)) = g(e.map(e.inv(y)))
            order_iso_right_inverse(e, y)
            e.map(e.inv(y)) = y
            f(e.map(e.inv(y))) = f(y)
            g(e.map(e.inv(y))) = g(y)
            f(y) <= g(y)
        }
        function_order_lte(f, g) = forall(y: Y) {
            f(y) <= g(y)
        }
        function_order_lte(f, g)
    }
}

/// Reindexing by an order isomorphism preserves and reflects pointwise order.
theorem function_order_iso_reindex_lte_iff[X: PartialOrder, Y: PartialOrder, A: PartialOrder](
    e: OrderIso[X, Y],
    f: Y -> A,
    g: Y -> A
) {
    function_order_lte(function_order_iso_reindex(e, f), function_order_iso_reindex(e, g)) =
    function_order_lte(f, g)
} by {
    if function_order_lte(function_order_iso_reindex(e, f), function_order_iso_reindex(e, g)) {
        function_order_iso_reindex_reflects_lte(e, f, g)
        function_order_lte(f, g)
    }
    if function_order_lte(f, g) {
        function_order_iso_reindex_lte(e, f, g)
        function_order_lte(function_order_iso_reindex(e, f), function_order_iso_reindex(e, g))
    }
    function_order_lte(function_order_iso_reindex(e, f), function_order_iso_reindex(e, g)) =
        function_order_lte(f, g)
}

/// Reindexing by the inverse of an order isomorphism preserves pointwise order.
theorem function_order_iso_unreindex_lte[X: PartialOrder, Y: PartialOrder, A: PartialOrder](
    e: OrderIso[X, Y],
    f: X -> A,
    g: X -> A
) {
    function_order_lte(f, g) implies
    function_order_lte(function_order_iso_unreindex(e, f), function_order_iso_unreindex(e, g))
} by {
    if function_order_lte(f, g) {
        function_order_lte(f, g) = forall(x: X) {
            f(x) <= g(x)
        }
        forall(y: Y) {
            f(e.inv(y)) <= g(e.inv(y))
            function_order_iso_unreindex(e, f, y) = f(e.inv(y))
            function_order_iso_unreindex(e, g, y) = g(e.inv(y))
            function_order_iso_unreindex(e, f, y) <= function_order_iso_unreindex(e, g, y)
        }
        function_order_lte(function_order_iso_unreindex(e, f), function_order_iso_unreindex(e, g)) =
            forall(y: Y) {
                function_order_iso_unreindex(e, f, y) <= function_order_iso_unreindex(e, g, y)
            }
        function_order_lte(function_order_iso_unreindex(e, f), function_order_iso_unreindex(e, g))
    }
}

/// Reindexing by the inverse of an order isomorphism reflects pointwise order.
theorem function_order_iso_unreindex_reflects_lte[X: PartialOrder, Y: PartialOrder, A: PartialOrder](
    e: OrderIso[X, Y],
    f: X -> A,
    g: X -> A
) {
    function_order_lte(function_order_iso_unreindex(e, f), function_order_iso_unreindex(e, g))
    implies
    function_order_lte(f, g)
} by {
    if function_order_lte(function_order_iso_unreindex(e, f), function_order_iso_unreindex(e, g)) {
        function_order_lte(function_order_iso_unreindex(e, f), function_order_iso_unreindex(e, g)) =
            forall(y: Y) {
                function_order_iso_unreindex(e, f, y) <= function_order_iso_unreindex(e, g, y)
            }
        forall(x: X) {
            function_order_iso_unreindex(e, f, e.map(x)) <= function_order_iso_unreindex(e, g, e.map(x))
            function_order_iso_unreindex(e, f, e.map(x)) = f(e.inv(e.map(x)))
            function_order_iso_unreindex(e, g, e.map(x)) = g(e.inv(e.map(x)))
            order_iso_left_inverse(e, x)
            e.inv(e.map(x)) = x
            f(e.inv(e.map(x))) = f(x)
            g(e.inv(e.map(x))) = g(x)
            f(x) <= g(x)
        }
        function_order_lte(f, g) = forall(x: X) {
            f(x) <= g(x)
        }
        function_order_lte(f, g)
    }
}

/// Reindexing by the inverse of an order isomorphism preserves and reflects pointwise order.
theorem function_order_iso_unreindex_lte_iff[X: PartialOrder, Y: PartialOrder, A: PartialOrder](
    e: OrderIso[X, Y],
    f: X -> A,
    g: X -> A
) {
    function_order_lte(function_order_iso_unreindex(e, f), function_order_iso_unreindex(e, g)) =
    function_order_lte(f, g)
} by {
    if function_order_lte(function_order_iso_unreindex(e, f), function_order_iso_unreindex(e, g)) {
        function_order_iso_unreindex_reflects_lte(e, f, g)
        function_order_lte(f, g)
    }
    if function_order_lte(f, g) {
        function_order_iso_unreindex_lte(e, f, g)
        function_order_lte(function_order_iso_unreindex(e, f), function_order_iso_unreindex(e, g))
    }
    function_order_lte(function_order_iso_unreindex(e, f), function_order_iso_unreindex(e, g)) =
        function_order_lte(f, g)
}

/// Reindexing by an order isomorphism preserves pointwise strict order.
theorem function_order_iso_reindex_lt[X: PartialOrder, Y: PartialOrder, A: PartialOrder](
    e: OrderIso[X, Y],
    f: Y -> A,
    g: Y -> A
) {
    function_order_lt(f, g) implies
    function_order_lt(function_order_iso_reindex(e, f), function_order_iso_reindex(e, g))
} by {
    if function_order_lt(f, g) {
        function_order_lt(f, g) = forall(y: Y) {
            f(y) < g(y)
        }
        forall(x: X) {
            f(e.map(x)) < g(e.map(x))
            function_order_iso_reindex(e, f, x) = f(e.map(x))
            function_order_iso_reindex(e, g, x) = g(e.map(x))
            function_order_iso_reindex(e, f, x) < function_order_iso_reindex(e, g, x)
        }
        function_order_lt(function_order_iso_reindex(e, f), function_order_iso_reindex(e, g)) =
            forall(x: X) {
                function_order_iso_reindex(e, f, x) < function_order_iso_reindex(e, g, x)
            }
        function_order_lt(function_order_iso_reindex(e, f), function_order_iso_reindex(e, g))
    }
}

/// Reindexing by an order isomorphism reflects pointwise strict order.
theorem function_order_iso_reindex_reflects_lt[X: PartialOrder, Y: PartialOrder, A: PartialOrder](
    e: OrderIso[X, Y],
    f: Y -> A,
    g: Y -> A
) {
    function_order_lt(function_order_iso_reindex(e, f), function_order_iso_reindex(e, g))
    implies
    function_order_lt(f, g)
} by {
    if function_order_lt(function_order_iso_reindex(e, f), function_order_iso_reindex(e, g)) {
        function_order_lt(function_order_iso_reindex(e, f), function_order_iso_reindex(e, g)) =
            forall(x: X) {
                function_order_iso_reindex(e, f, x) < function_order_iso_reindex(e, g, x)
            }
        forall(y: Y) {
            function_order_iso_reindex(e, f, e.inv(y)) < function_order_iso_reindex(e, g, e.inv(y))
            function_order_iso_reindex(e, f, e.inv(y)) = f(e.map(e.inv(y)))
            function_order_iso_reindex(e, g, e.inv(y)) = g(e.map(e.inv(y)))
            order_iso_right_inverse(e, y)
            e.map(e.inv(y)) = y
            f(e.map(e.inv(y))) = f(y)
            g(e.map(e.inv(y))) = g(y)
            f(y) < g(y)
        }
        function_order_lt(f, g) = forall(y: Y) {
            f(y) < g(y)
        }
        function_order_lt(f, g)
    }
}

/// Reindexing by an order isomorphism preserves and reflects pointwise strict order.
theorem function_order_iso_reindex_lt_iff[X: PartialOrder, Y: PartialOrder, A: PartialOrder](
    e: OrderIso[X, Y],
    f: Y -> A,
    g: Y -> A
) {
    function_order_lt(function_order_iso_reindex(e, f), function_order_iso_reindex(e, g)) =
    function_order_lt(f, g)
} by {
    if function_order_lt(function_order_iso_reindex(e, f), function_order_iso_reindex(e, g)) {
        function_order_iso_reindex_reflects_lt(e, f, g)
        function_order_lt(f, g)
    }
    if function_order_lt(f, g) {
        function_order_iso_reindex_lt(e, f, g)
        function_order_lt(function_order_iso_reindex(e, f), function_order_iso_reindex(e, g))
    }
    function_order_lt(function_order_iso_reindex(e, f), function_order_iso_reindex(e, g)) =
        function_order_lt(f, g)
}

/// Reindexing by the inverse of an order isomorphism preserves pointwise strict order.
theorem function_order_iso_unreindex_lt[X: PartialOrder, Y: PartialOrder, A: PartialOrder](
    e: OrderIso[X, Y],
    f: X -> A,
    g: X -> A
) {
    function_order_lt(f, g) implies
    function_order_lt(function_order_iso_unreindex(e, f), function_order_iso_unreindex(e, g))
} by {
    if function_order_lt(f, g) {
        function_order_lt(f, g) = forall(x: X) {
            f(x) < g(x)
        }
        forall(y: Y) {
            f(e.inv(y)) < g(e.inv(y))
            function_order_iso_unreindex(e, f, y) = f(e.inv(y))
            function_order_iso_unreindex(e, g, y) = g(e.inv(y))
            function_order_iso_unreindex(e, f, y) < function_order_iso_unreindex(e, g, y)
        }
        function_order_lt(function_order_iso_unreindex(e, f), function_order_iso_unreindex(e, g)) =
            forall(y: Y) {
                function_order_iso_unreindex(e, f, y) < function_order_iso_unreindex(e, g, y)
            }
        function_order_lt(function_order_iso_unreindex(e, f), function_order_iso_unreindex(e, g))
    }
}

/// Reindexing by the inverse of an order isomorphism reflects pointwise strict order.
theorem function_order_iso_unreindex_reflects_lt[X: PartialOrder, Y: PartialOrder, A: PartialOrder](
    e: OrderIso[X, Y],
    f: X -> A,
    g: X -> A
) {
    function_order_lt(function_order_iso_unreindex(e, f), function_order_iso_unreindex(e, g))
    implies
    function_order_lt(f, g)
} by {
    if function_order_lt(function_order_iso_unreindex(e, f), function_order_iso_unreindex(e, g)) {
        function_order_lt(function_order_iso_unreindex(e, f), function_order_iso_unreindex(e, g)) =
            forall(y: Y) {
                function_order_iso_unreindex(e, f, y) < function_order_iso_unreindex(e, g, y)
            }
        forall(x: X) {
            function_order_iso_unreindex(e, f, e.map(x)) < function_order_iso_unreindex(e, g, e.map(x))
            function_order_iso_unreindex(e, f, e.map(x)) = f(e.inv(e.map(x)))
            function_order_iso_unreindex(e, g, e.map(x)) = g(e.inv(e.map(x)))
            order_iso_left_inverse(e, x)
            e.inv(e.map(x)) = x
            f(e.inv(e.map(x))) = f(x)
            g(e.inv(e.map(x))) = g(x)
            f(x) < g(x)
        }
        function_order_lt(f, g) = forall(x: X) {
            f(x) < g(x)
        }
        function_order_lt(f, g)
    }
}

/// Reindexing by the inverse of an order isomorphism preserves and reflects pointwise strict order.
theorem function_order_iso_unreindex_lt_iff[X: PartialOrder, Y: PartialOrder, A: PartialOrder](
    e: OrderIso[X, Y],
    f: X -> A,
    g: X -> A
) {
    function_order_lt(function_order_iso_unreindex(e, f), function_order_iso_unreindex(e, g)) =
    function_order_lt(f, g)
} by {
    if function_order_lt(function_order_iso_unreindex(e, f), function_order_iso_unreindex(e, g)) {
        function_order_iso_unreindex_reflects_lt(e, f, g)
        function_order_lt(f, g)
    }
    if function_order_lt(f, g) {
        function_order_iso_unreindex_lt(e, f, g)
        function_order_lt(function_order_iso_unreindex(e, f), function_order_iso_unreindex(e, g))
    }
    function_order_lt(function_order_iso_unreindex(e, f), function_order_iso_unreindex(e, g)) =
        function_order_lt(f, g)
}

/// An order isomorphism pair is a Galois connection.
theorem order_iso_pair_is_galois_connection[A: PartialOrder, B: PartialOrder](
    f: A -> B,
    g: B -> A
) {
    is_order_iso_pair(f, g) implies is_galois_connection(f, g)
} by {
    if is_order_iso_pair(f, g) {
        order_iso_pair_map_is_order_embedding(f, g)
        order_iso_pair_inv_is_order_embedding(f, g)
        is_order_embedding(f)
        is_order_embedding(g)
        forall(a: A, b: B) {
            order_iso_pair_right_inverse(f, g, b)
            f(g(b)) = b
            order_iso_pair_left_inverse(f, g, a)
            g(f(a)) = a
            if f(a) <= b {
                f(a) <= f(g(b))
                order_embedding_reflects_lte(f, a, g(b))
                a <= g(b)
            }
            if a <= g(b) {
                g(f(a)) <= g(b)
                order_embedding_reflects_lte(g, f(a), b)
                f(a) <= b
            }
            f(a) <= b = (a <= g(b))
        }
    }
}

/// An order isomorphism pair is a Galois insertion.
theorem order_iso_pair_is_galois_insertion[A: PartialOrder, B: PartialOrder](
    f: A -> B,
    g: B -> A
) {
    is_order_iso_pair(f, g) implies is_galois_insertion(f, g)
} by {
    if is_order_iso_pair(f, g) {
        order_iso_pair_is_galois_connection(f, g)
        is_galois_connection(f, g)
        forall(b: B) {
            order_iso_pair_right_inverse(f, g, b)
            f(g(b)) = b
        }
        is_galois_insertion(f, g) = (is_galois_connection(f, g) and forall(b: B) {
            f(g(b)) = b
        })
        is_galois_insertion(f, g)
    }
}

/// An order isomorphism pair is a Galois coinsertion.
theorem order_iso_pair_is_galois_coinsertion[A: PartialOrder, B: PartialOrder](
    f: A -> B,
    g: B -> A
) {
    is_order_iso_pair(f, g) implies is_galois_coinsertion(f, g)
} by {
    if is_order_iso_pair(f, g) {
        order_iso_pair_is_galois_connection(f, g)
        is_galois_connection(f, g)
        forall(a: A) {
            order_iso_pair_left_inverse(f, g, a)
            g(f(a)) = a
        }
        is_galois_coinsertion(f, g) = (is_galois_connection(f, g) and forall(a: A) {
            g(f(a)) = a
        })
        is_galois_coinsertion(f, g)
    }
}
