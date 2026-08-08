from algebra.add_comm_group import AddCommGroup
from affine_space import AffineSpace, affine_vsub_self, affine_zero_vadd
from affine_subspace import AffineSubspace, affine_subspace_closed,
    affine_subspace_constraint, affine_subspace_subset, affine_subspace_ext

/// True if a function between affine spaces preserves the affine
/// combination `p - q + r`.
define affine_map_constraint[V1: AddCommGroup, P1, V2: AddCommGroup, P2](
    s: AffineSpace[V1, P1], t: AffineSpace[V2, P2], to_fun: P1 -> P2) -> Bool {
    forall(p: P1, q: P1, r: P1) {
        to_fun(s.vadd(s.vsub(p, q), r)) = t.vadd(t.vsub(to_fun(p), to_fun(q)), to_fun(r))
    }
}

/// An affine map between two affine spaces: a function preserving the
/// affine combination `p - q + r`.
structure AffineMap[V1: AddCommGroup, P1, V2: AddCommGroup, P2] {
    /// The source affine space.
    src: AffineSpace[V1, P1]
    /// The target affine space.
    dst: AffineSpace[V2, P2]
    /// The underlying function on points.
    to_fun: P1 -> P2
} constraint {
    affine_map_constraint(src, dst, to_fun)
}

/// Evaluation of an affine map constructed from specified data.
theorem affine_map_new_apply[V1: AddCommGroup, P1, V2: AddCommGroup, P2](
    s: AffineSpace[V1, P1], t: AffineSpace[V2, P2], to_fun: P1 -> P2,
    f: AffineMap[V1, P1, V2, P2], p: P1
) {
    AffineMap.new(s, t, to_fun) = Option.some(f) implies f.to_fun(p) = to_fun(p)
} by {
    if AffineMap.new(s, t, to_fun) = Option.some(f) {
        f.to_fun(p) = to_fun(p)
    }
}

/// The source of an affine map constructed from specified data.
theorem affine_map_new_src[V1: AddCommGroup, P1, V2: AddCommGroup, P2](
    s: AffineSpace[V1, P1], t: AffineSpace[V2, P2], to_fun: P1 -> P2,
    f: AffineMap[V1, P1, V2, P2]
) {
    AffineMap.new(s, t, to_fun) = Option.some(f) implies f.src = s
} by {
    if AffineMap.new(s, t, to_fun) = Option.some(f) {
        f.src = s
    }
}

/// The target of an affine map constructed from specified data.
theorem affine_map_new_dst[V1: AddCommGroup, P1, V2: AddCommGroup, P2](
    s: AffineSpace[V1, P1], t: AffineSpace[V2, P2], to_fun: P1 -> P2,
    f: AffineMap[V1, P1, V2, P2]
) {
    AffineMap.new(s, t, to_fun) = Option.some(f) implies f.dst = t
} by {
    if AffineMap.new(s, t, to_fun) = Option.some(f) {
        f.dst = t
    }
}

/// Affine map extensionality from agreement of source, target, and underlying function.
theorem affine_map_ext[V1: AddCommGroup, P1, V2: AddCommGroup, P2](
    f: AffineMap[V1, P1, V2, P2],
    g: AffineMap[V1, P1, V2, P2]) {
    f.src = g.src and f.dst = g.dst
        and (forall(x: P1) { f.to_fun(x) = g.to_fun(x) })
        implies f = g
} by {
    if f.src = g.src and f.dst = g.dst
        and (forall(x: P1) { f.to_fun(x) = g.to_fun(x) }) {
        f.to_fun = g.to_fun
    }
}

/// An affine map preserves the affine combination `p - q + r`.
theorem affine_map_preserves[V1: AddCommGroup, P1, V2: AddCommGroup, P2](
    f: AffineMap[V1, P1, V2, P2], p: P1, q: P1, r: P1) {
    f.to_fun(f.src.vadd(f.src.vsub(p, q), r)) =
        f.dst.vadd(f.dst.vsub(f.to_fun(p), f.to_fun(q)), f.to_fun(r))
} by {
    affine_map_constraint[V1, P1, V2, P2](f.src, f.dst, f.to_fun) = forall(x: P1, y: P1, z: P1) {
        f.to_fun(f.src.vadd(f.src.vsub(x, y), z)) =
            f.dst.vadd(f.dst.vsub(f.to_fun(x), f.to_fun(y)), f.to_fun(z))
    }
}

/// The identity function on points satisfies the affine-map closure property.
theorem affine_map_id_constraint[V: AddCommGroup, P](a: AffineSpace[V, P]) {
    affine_map_constraint(a, a, function(x: P) { x })
} by {
    let id: P -> P = function(x: P) { x }
    forall(p: P, q: P, r: P) {
        a.vadd(a.vsub(id(p), id(q)), id(r)) = a.vadd(a.vsub(p, q), r)
    }
}

/// The identity affine map of an affine space.
let affine_map_id[V: AddCommGroup, P](a: AffineSpace[V, P]) -> result: AffineMap[V, P, V, P] satisfy {
    AffineMap[V, P, V, P].new(a, a, function(x: P) { x }) = Option.some(result)
} by {
    affine_map_id_constraint(a)
}

/// The identity affine map evaluates to its argument.
theorem affine_map_id_apply[V: AddCommGroup, P](a: AffineSpace[V, P], x: P) {
    affine_map_id(a).to_fun(x) = x
} by {
    affine_map_id(a).to_fun = function(y: P) { y }
}

/// The identity affine map's source equals the given affine space.
theorem affine_map_id_src[V: AddCommGroup, P](a: AffineSpace[V, P]) {
    affine_map_id(a).src = a
}

/// The identity affine map's target equals the given affine space.
theorem affine_map_id_dst[V: AddCommGroup, P](a: AffineSpace[V, P]) {
    affine_map_id(a).dst = a
}

/// A constant function between affine spaces satisfies the affine-map closure property.
theorem affine_map_const_constraint[V1: AddCommGroup, P1, V2: AddCommGroup, P2](s: AffineSpace[V1, P1], t: AffineSpace[V2, P2], c: P2) {
    affine_map_constraint(s, t, function(x: P1) { c })
} by {
    let k: P1 -> P2 = function(x: P1) { c }
    forall(p: P1, q: P1, r: P1) {
        affine_vsub_self(t, c)
        affine_zero_vadd(t, c)
        k(s.vadd(s.vsub(p, q), r)) = t.vadd(t.vsub(k(p), k(q)), k(r))
    }
}

/// The constant affine map sending every point of `s` to `c`.
let affine_map_const[V1: AddCommGroup, P1, V2: AddCommGroup, P2](s: AffineSpace[V1, P1], t: AffineSpace[V2, P2], c: P2) -> result: AffineMap[V1, P1, V2, P2] satisfy {
    AffineMap[V1, P1, V2, P2].new(s, t, function(x: P1) { c }) = Option.some(result)
} by {
    affine_map_const_constraint(s, t, c)
}

/// The constant affine map evaluates to its constant value.
theorem affine_map_const_apply[V1: AddCommGroup, P1, V2: AddCommGroup, P2](s: AffineSpace[V1, P1], t: AffineSpace[V2, P2], c: P2, x: P1) {
    affine_map_const(s, t, c).to_fun(x) = c
} by {
    affine_map_const(s, t, c).to_fun = function(y: P1) { c }
}

/// The constant affine map's source equals the given source space.
theorem affine_map_const_src[V1: AddCommGroup, P1, V2: AddCommGroup, P2](s: AffineSpace[V1, P1], t: AffineSpace[V2, P2], c: P2) {
    affine_map_const(s, t, c).src = s
}

/// The constant affine map's target equals the given target space.
theorem affine_map_const_dst[V1: AddCommGroup, P1, V2: AddCommGroup, P2](s: AffineSpace[V1, P1], t: AffineSpace[V2, P2], c: P2) {
    affine_map_const(s, t, c).dst = t
}

/// The composition function on points satisfies the affine combination identity pointwise.
theorem affine_map_compose_pointwise[V1: AddCommGroup, P1, V2: AddCommGroup, P2, V3: AddCommGroup, P3](f: AffineMap[V1, P1, V2, P2], g: AffineMap[V2, P2, V3, P3], p: P1, q: P1, r: P1) {
    f.dst = g.src implies g.to_fun(f.to_fun(f.src.vadd(f.src.vsub(p, q), r))) = g.dst.vadd(g.dst.vsub(g.to_fun(f.to_fun(p)), g.to_fun(f.to_fun(q))), g.to_fun(f.to_fun(r)))
} by {
    if f.dst = g.src {
        let mp: P2 = f.to_fun(p)
        let mq: P2 = f.to_fun(q)
        let mr: P2 = f.to_fun(r)
        affine_map_preserves(f, p, q, r)
        f.dst.vadd(f.dst.vsub(mp, mq), mr) = g.src.vadd(g.src.vsub(mp, mq), mr)
        affine_map_preserves(g, mp, mq, mr)
    }
}

/// When the target of the first map equals the source of the second, the composed point-function satisfies the affine-map closure property.
theorem affine_map_compose_constraint[V1: AddCommGroup, P1, V2: AddCommGroup, P2, V3: AddCommGroup, P3](f: AffineMap[V1, P1, V2, P2], g: AffineMap[V2, P2, V3, P3]) {
    f.dst = g.src implies affine_map_constraint(f.src, g.dst, function(x: P1) { g.to_fun(f.to_fun(x)) })
} by {
    if f.dst = g.src {
        let h: P1 -> P3 = function(x: P1) { g.to_fun(f.to_fun(x)) }
        forall(p: P1, q: P1, r: P1) {
            affine_map_compose_pointwise(f, g, p, q, r)
            h(f.src.vadd(f.src.vsub(p, q), r)) = g.dst.vadd(g.dst.vsub(h(p), h(q)), h(r))
        }
    }
}

/// The composition of two affine maps, returning `Option.some` exactly when the target of the first equals the source of the second.
define affine_map_compose[V1: AddCommGroup, P1, V2: AddCommGroup, P2, V3: AddCommGroup, P3](f: AffineMap[V1, P1, V2, P2], g: AffineMap[V2, P2, V3, P3]) -> Option[AffineMap[V1, P1, V3, P3]] {
    AffineMap[V1, P1, V3, P3].new(f.src, g.dst, function(x: P1) { g.to_fun(f.to_fun(x)) })
}

/// When the target of the first map equals the source of the second, the composition is defined.
theorem affine_map_compose_some[V1: AddCommGroup, P1, V2: AddCommGroup, P2, V3: AddCommGroup, P3](f: AffineMap[V1, P1, V2, P2], g: AffineMap[V2, P2, V3, P3]) {
    f.dst = g.src implies exists(h: AffineMap[V1, P1, V3, P3]) {
        affine_map_compose(f, g) = Option.some(h)
    }
} by {
    if f.dst = g.src {
        affine_map_compose_constraint(f, g)
        let composed: AffineMap[V1, P1, V3, P3] satisfy {
            AffineMap[V1, P1, V3, P3].new(f.src, g.dst, function(x: P1) { g.to_fun(f.to_fun(x)) }) = Option.some(composed)
        }
        affine_map_compose(f, g) = Option.some(composed)
    }
}

/// If the composition is some bundled map, its source is the source of the first map.
theorem affine_map_compose_src[V1: AddCommGroup, P1, V2: AddCommGroup, P2, V3: AddCommGroup, P3](f: AffineMap[V1, P1, V2, P2], g: AffineMap[V2, P2, V3, P3], h: AffineMap[V1, P1, V3, P3]) {
    affine_map_compose(f, g) = Option.some(h) implies h.src = f.src
}

/// If the composition is some bundled map, its target is the target of the second map.
theorem affine_map_compose_dst[V1: AddCommGroup, P1, V2: AddCommGroup, P2, V3: AddCommGroup, P3](f: AffineMap[V1, P1, V2, P2], g: AffineMap[V2, P2, V3, P3], h: AffineMap[V1, P1, V3, P3]) {
    affine_map_compose(f, g) = Option.some(h) implies h.dst = g.dst
}

/// If the composition is some bundled map, it evaluates as the iterated application.
theorem affine_map_compose_apply[V1: AddCommGroup, P1, V2: AddCommGroup, P2, V3: AddCommGroup, P3](f: AffineMap[V1, P1, V2, P2], g: AffineMap[V2, P2, V3, P3], h: AffineMap[V1, P1, V3, P3], x: P1) {
    affine_map_compose(f, g) = Option.some(h) implies h.to_fun(x) = g.to_fun(f.to_fun(x))
} by {
    if affine_map_compose(f, g) = Option.some(h) {
        AffineMap[V1, P1, V3, P3].new(f.src, g.dst, function(y: P1) { g.to_fun(f.to_fun(y)) }) = Option.some(h)
        h.to_fun = function(y: P1) { g.to_fun(f.to_fun(y)) }
    }
}

/// The membership predicate of the preimage of an affine subspace under an
/// affine map: a point `x` of the source belongs when `f.to_fun(x)` lies in `s`.
define affine_map_preimage_contains[V1: AddCommGroup, P1, V2: AddCommGroup, P2](
    f: AffineMap[V1, P1, V2, P2], s: AffineSubspace[V2, P2], x: P1) -> Bool {
    s.contains(f.to_fun(x))
}

/// The preimage predicate satisfies the affine-subspace closure property on the
/// source affine space when the target subspace is over the map's target space.
theorem affine_map_preimage_constraint[V1: AddCommGroup, P1, V2: AddCommGroup, P2](
    f: AffineMap[V1, P1, V2, P2], s: AffineSubspace[V2, P2]) {
    s.space = f.dst implies affine_subspace_constraint(f.src,
        affine_map_preimage_contains(f, s))
} by {
    if s.space = f.dst {
        forall(p: P1, q: P1, r: P1) {
            if affine_map_preimage_contains(f, s, p)
               and affine_map_preimage_contains(f, s, q)
               and affine_map_preimage_contains(f, s, r) {
                s.contains(f.to_fun(p))
                s.contains(f.to_fun(q))
                s.contains(f.to_fun(r))
                affine_subspace_closed(s, f.to_fun(p), f.to_fun(q), f.to_fun(r))
                affine_map_preserves(f, p, q, r)
                affine_map_preimage_contains(f, s, f.src.vadd(f.src.vsub(p, q), r))
            }
        }
    }
}

/// The preimage of an affine subspace under an affine map; `Option.some` exactly
/// when the subspace lives in the map's target affine space.
define affine_map_preimage[V1: AddCommGroup, P1, V2: AddCommGroup, P2](
    f: AffineMap[V1, P1, V2, P2], s: AffineSubspace[V2, P2]) -> Option[AffineSubspace[V1, P1]] {
    AffineSubspace[V1, P1].new(f.src,
        affine_map_preimage_contains(f, s))
}

/// When the subspace lives in the map's target space, its preimage is defined.
theorem affine_map_preimage_some[V1: AddCommGroup, P1, V2: AddCommGroup, P2](
    f: AffineMap[V1, P1, V2, P2], s: AffineSubspace[V2, P2]) {
    s.space = f.dst implies exists(t: AffineSubspace[V1, P1]) {
        affine_map_preimage(f, s) = Option.some(t)
    }
} by {
    if s.space = f.dst {
        affine_map_preimage_constraint(f, s)
        let pre: AffineSubspace[V1, P1] satisfy {
            AffineSubspace[V1, P1].new(f.src,
                affine_map_preimage_contains(f, s)) = Option.some(pre)
        }
        affine_map_preimage(f, s) = Option.some(pre)
    }
}

/// If the preimage is some bundled subspace, its ambient space is the map's source.
theorem affine_map_preimage_space[V1: AddCommGroup, P1, V2: AddCommGroup, P2](
    f: AffineMap[V1, P1, V2, P2], s: AffineSubspace[V2, P2], t: AffineSubspace[V1, P1]) {
    affine_map_preimage(f, s) = Option.some(t) implies t.space = f.src
}

/// If the preimage is some bundled subspace, a point lies in it exactly when its
/// image lies in the original subspace.
theorem affine_map_preimage_contains_iff[V1: AddCommGroup, P1, V2: AddCommGroup, P2](
    f: AffineMap[V1, P1, V2, P2], s: AffineSubspace[V2, P2], t: AffineSubspace[V1, P1], x: P1) {
    affine_map_preimage(f, s) = Option.some(t) implies
        (t.contains(x) = s.contains(f.to_fun(x)))
} by {
    if affine_map_preimage(f, s) = Option.some(t) {
        AffineSubspace[V1, P1].new(f.src,
            function(y: P1) { affine_map_preimage_contains(f, s, y) }) = Option.some(t)
        t.contains = function(y: P1) { affine_map_preimage_contains(f, s, y) }
    }
}

/// If an affine map sends a source point to `y`, membership of that point in a
/// preimage is membership of `y` in the target subspace.
theorem affine_map_preimage_contains_of_apply_eq[V1: AddCommGroup, P1, V2: AddCommGroup, P2](
    f: AffineMap[V1, P1, V2, P2],
    s: AffineSubspace[V2, P2],
    t: AffineSubspace[V1, P1],
    x: P1,
    y: P2
) {
    affine_map_preimage(f, s) = Option.some(t)
        and f.to_fun(x) = y
        implies t.contains(x) = s.contains(y)
} by {
    if affine_map_preimage(f, s) = Option.some(t)
        and f.to_fun(x) = y {
        affine_map_preimage_contains_iff(f, s, t, x)
    }
}

/// A bundled preimage has the same membership predicate as any subspace whose
/// membership is pointwise membership of the image.
theorem affine_map_preimage_same_contains[V1: AddCommGroup, P1, V2: AddCommGroup, P2](
    f: AffineMap[V1, P1, V2, P2],
    s: AffineSubspace[V2, P2],
    t: AffineSubspace[V1, P1],
    u: AffineSubspace[V1, P1]
) {
    affine_map_preimage(f, s) = Option.some(t)
        and (forall(x: P1) {
            u.contains(x) = s.contains(f.to_fun(x))
        })
        implies forall(x: P1) {
            t.contains(x) = u.contains(x)
        }
} by {
    if affine_map_preimage(f, s) = Option.some(t)
        and (forall(x: P1) {
            u.contains(x) = s.contains(f.to_fun(x))
        }) {
        forall(x: P1) {
            affine_map_preimage_contains_iff(f, s, t, x)
            t.contains(x) = u.contains(x)
        }
    }
}

/// A bundled preimage equals any affine subspace with the same ambient space
/// and the same pointwise membership predicate.
theorem affine_map_preimage_eq_of_contains[V1: AddCommGroup, P1, V2: AddCommGroup, P2](
    f: AffineMap[V1, P1, V2, P2],
    s: AffineSubspace[V2, P2],
    t: AffineSubspace[V1, P1],
    u: AffineSubspace[V1, P1]
) {
    affine_map_preimage(f, s) = Option.some(t)
        and t.space = u.space
        and (forall(x: P1) {
            u.contains(x) = s.contains(f.to_fun(x))
        })
        implies t = u
} by {
    if affine_map_preimage(f, s) = Option.some(t)
        and t.space = u.space
        and (forall(x: P1) {
            u.contains(x) = s.contains(f.to_fun(x))
        }) {
        affine_map_preimage_same_contains(f, s, t, u)
        forall(x: P1) {
            t.contains(x) = u.contains(x)
        }
        affine_subspace_ext(t, u)
        t = u
    }
}

/// Equal affine subspaces may be substituted in a bundled preimage.
theorem affine_map_preimage_some_of_eq[V1: AddCommGroup, P1, V2: AddCommGroup, P2](
    f: AffineMap[V1, P1, V2, P2],
    s: AffineSubspace[V2, P2],
    t: AffineSubspace[V1, P1],
    u: AffineSubspace[V1, P1]
) {
    affine_map_preimage(f, s) = Option.some(t)
        and t = u
        implies affine_map_preimage(f, s) = Option.some(u)
} by {
    if affine_map_preimage(f, s) = Option.some(t)
        and t = u {
    }
}

/// The membership predicate of the image of an affine subspace under an
/// affine map: a point `y` of the target belongs when some point of the
/// source subspace maps to it.
define affine_map_image_contains[V1: AddCommGroup, P1, V2: AddCommGroup, P2](
    f: AffineMap[V1, P1, V2, P2], s: AffineSubspace[V1, P1], y: P2) -> Bool {
    exists(x: P1) {
        s.contains(x) and f.to_fun(x) = y
    }
}

/// The image predicate satisfies the affine-subspace closure property on the
/// target affine space when the source subspace lives in the map's source space.
theorem affine_map_image_constraint[V1: AddCommGroup, P1, V2: AddCommGroup, P2](
    f: AffineMap[V1, P1, V2, P2], s: AffineSubspace[V1, P1]) {
    s.space = f.src implies affine_subspace_constraint(f.dst,
        affine_map_image_contains(f, s))
} by {
    if s.space = f.src {
        forall(p2: P2, q2: P2, r2: P2) {
            if affine_map_image_contains(f, s, p2)
               and affine_map_image_contains(f, s, q2)
               and affine_map_image_contains(f, s, r2) {
                let p: P1 satisfy {
                    s.contains(p) and f.to_fun(p) = p2
                }
                let q: P1 satisfy {
                    s.contains(q) and f.to_fun(q) = q2
                }
                let r: P1 satisfy {
                    s.contains(r) and f.to_fun(r) = r2
                }
                affine_subspace_closed(s, p, q, r)
                let w: P1 = s.space.vadd(s.space.vsub(p, q), r)
                s.contains(w)
                s.space.vadd(s.space.vsub(p, q), r) = f.src.vadd(f.src.vsub(p, q), r)
                w = f.src.vadd(f.src.vsub(p, q), r)
                affine_map_preserves(f, p, q, r)
                f.to_fun(f.src.vadd(f.src.vsub(p, q), r)) =
                    f.dst.vadd(f.dst.vsub(f.to_fun(p), f.to_fun(q)), f.to_fun(r))
                f.to_fun(w) = f.dst.vadd(f.dst.vsub(p2, q2), r2)
                affine_map_image_contains(f, s, f.dst.vadd(f.dst.vsub(p2, q2), r2))
            }
        }
    }
}

/// The image of an affine subspace under an affine map; `Option.some` exactly
/// when the subspace lives in the map's source affine space.
define affine_map_image[V1: AddCommGroup, P1, V2: AddCommGroup, P2](
    f: AffineMap[V1, P1, V2, P2], s: AffineSubspace[V1, P1]) -> Option[AffineSubspace[V2, P2]] {
    AffineSubspace[V2, P2].new(f.dst,
        affine_map_image_contains(f, s))
}

/// When the subspace lives in the map's source space, its image is defined.
theorem affine_map_image_some[V1: AddCommGroup, P1, V2: AddCommGroup, P2](
    f: AffineMap[V1, P1, V2, P2], s: AffineSubspace[V1, P1]) {
    s.space = f.src implies exists(t: AffineSubspace[V2, P2]) {
        affine_map_image(f, s) = Option.some(t)
    }
} by {
    if s.space = f.src {
        affine_map_image_constraint(f, s)
        let img: AffineSubspace[V2, P2] satisfy {
            AffineSubspace[V2, P2].new(f.dst,
                affine_map_image_contains(f, s)) = Option.some(img)
        }
        affine_map_image(f, s) = Option.some(img)
    }
}

/// If the image is some bundled subspace, its ambient space is the map's target.
theorem affine_map_image_space[V1: AddCommGroup, P1, V2: AddCommGroup, P2](
    f: AffineMap[V1, P1, V2, P2], s: AffineSubspace[V1, P1], t: AffineSubspace[V2, P2]) {
    affine_map_image(f, s) = Option.some(t) implies t.space = f.dst
}

/// If the image is some bundled subspace, a point lies in it exactly when it
/// is the image of some point of the source subspace.
theorem affine_map_image_contains_iff[V1: AddCommGroup, P1, V2: AddCommGroup, P2](
    f: AffineMap[V1, P1, V2, P2], s: AffineSubspace[V1, P1], t: AffineSubspace[V2, P2], y: P2) {
    affine_map_image(f, s) = Option.some(t) implies
        (t.contains(y) = affine_map_image_contains(f, s, y))
} by {
    if affine_map_image(f, s) = Option.some(t) {
        AffineSubspace[V2, P2].new(f.dst,
            function(z: P2) { affine_map_image_contains(f, s, z) }) = Option.some(t)
        t.contains = function(z: P2) { affine_map_image_contains(f, s, z) }
    }
}

/// The image of any point of the source subspace lies in the image subspace.
theorem affine_map_image_contains_apply[V1: AddCommGroup, P1, V2: AddCommGroup, P2](
    f: AffineMap[V1, P1, V2, P2], s: AffineSubspace[V1, P1], t: AffineSubspace[V2, P2], x: P1) {
    affine_map_image(f, s) = Option.some(t) and s.contains(x)
        implies t.contains(f.to_fun(x))
} by {
    if affine_map_image(f, s) = Option.some(t) and s.contains(x) {
        affine_map_image_contains(f, s, f.to_fun(x))
        affine_map_image_contains_iff(f, s, t, f.to_fun(x))
    }
}

/// The image of an affine map is monotonic in the source subspace.
theorem affine_map_image_mono[V1: AddCommGroup, P1, V2: AddCommGroup, P2](
    f: AffineMap[V1, P1, V2, P2],
    s1: AffineSubspace[V1, P1],
    s2: AffineSubspace[V1, P1],
    t1: AffineSubspace[V2, P2],
    t2: AffineSubspace[V2, P2]) {
    affine_map_image(f, s1) = Option.some(t1)
        and affine_map_image(f, s2) = Option.some(t2)
        and affine_subspace_subset(s1, s2)
        implies affine_subspace_subset(t1, t2)
} by {
    if affine_map_image(f, s1) = Option.some(t1)
        and affine_map_image(f, s2) = Option.some(t2)
        and affine_subspace_subset(s1, s2) {
        affine_subspace_subset(s1, s2) = forall(z: P1) {
            s1.contains(z) implies s2.contains(z)
        }
        forall(y: P2) {
            if t1.contains(y) {
                affine_map_image_contains_iff(f, s1, t1, y)
                let x: P1 satisfy {
                    s1.contains(x) and f.to_fun(x) = y
                }
                affine_map_image_contains(f, s2, y)
                affine_map_image_contains_iff(f, s2, t2, y)
                t2.contains(y)
            }
        }
    }
}

/// The preimage of an affine map is monotonic in the target subspace.
theorem affine_map_preimage_mono[V1: AddCommGroup, P1, V2: AddCommGroup, P2](
    f: AffineMap[V1, P1, V2, P2],
    s1: AffineSubspace[V2, P2],
    s2: AffineSubspace[V2, P2],
    t1: AffineSubspace[V1, P1],
    t2: AffineSubspace[V1, P1]) {
    affine_map_preimage(f, s1) = Option.some(t1)
        and affine_map_preimage(f, s2) = Option.some(t2)
        and affine_subspace_subset(s1, s2)
        implies affine_subspace_subset(t1, t2)
} by {
    if affine_map_preimage(f, s1) = Option.some(t1)
        and affine_map_preimage(f, s2) = Option.some(t2)
        and affine_subspace_subset(s1, s2) {
        affine_subspace_subset(s1, s2) = forall(z: P2) {
            s1.contains(z) implies s2.contains(z)
        }
        forall(x: P1) {
            if t1.contains(x) {
                affine_map_preimage_contains_iff(f, s1, t1, x)
                affine_map_preimage_contains_iff(f, s2, t2, x)
                t2.contains(x)
            }
        }
    }
}

attributes AffineMap[V1: AddCommGroup, P1, V2: AddCommGroup, P2] {
    /// Affine-map extensionality from equality of source, target, and underlying function.
    let ext = affine_map_ext[V1, P1, V2, P2]
}
