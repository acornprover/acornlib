from data.basic.relation_basic import is_reflexive, is_transitive, is_antisymmetric, reflexive_self, transitive_step, antisymmetric_eq
from lte import LTE
from order import PartialOrder, LinearOrder, lte_refl, lte_trans, lte_antisymm, lte_or_gte

/// The opposite (dual) less-than-or-equal-to relation, with arguments reversed.
define opposite_lte[A: LTE](a: A, b: A) -> Bool {
    b <= a
}

/// The opposite relation is the original relation with reversed arguments.
theorem opposite_lte_eq[A: LTE](a: A, b: A) {
    opposite_lte(a, b) = b <= a
}

/// Reversing the arguments of the opposite relation recovers the original.
theorem opposite_lte_swap[A: LTE](a: A, b: A) {
    opposite_lte(b, a) = a <= b
}

/// The opposite of a reflexive relation is reflexive.
theorem opposite_lte_reflexive[A: LTE] {
    is_reflexive(A.lte) implies is_reflexive(opposite_lte[A])
} by {
    if is_reflexive(A.lte) {
        forall(a: A) {
            reflexive_self(A.lte, a)
            a <= a
            opposite_lte(a, a)
        }
    }
}

/// The opposite of a transitive relation is transitive.
theorem opposite_lte_transitive[A: LTE] {
    is_transitive(A.lte) implies is_transitive(opposite_lte[A])
} by {
    if is_transitive(A.lte) {
        forall(a: A, b: A, c: A) {
            if opposite_lte(a, b) and opposite_lte(b, c) {
                b <= a
                c <= b
                transitive_step(A.lte, c, b, a)
                c <= a
                opposite_lte(a, c)
            }
        }
    }
}

/// The opposite of an antisymmetric relation is antisymmetric.
theorem opposite_lte_antisymmetric[A: LTE] {
    is_antisymmetric(A.lte) implies is_antisymmetric(opposite_lte[A])
} by {
    if is_antisymmetric(A.lte) {
        forall(a: A, b: A) {
            if opposite_lte(a, b) and opposite_lte(b, a) {
                b <= a
                a <= b
                antisymmetric_eq(A.lte, a, b)
                a = b
            }
        }
    }
}

/// On a partial order, the opposite relation is reflexive.
theorem opposite_lte_refl[P: PartialOrder](a: P) {
    opposite_lte(a, a)
} by {
    lte_refl(a)
}

/// On a partial order, the opposite relation is transitive.
theorem opposite_lte_trans[P: PartialOrder](a: P, b: P, c: P) {
    opposite_lte(a, b) and opposite_lte(b, c) implies opposite_lte(a, c)
} by {
    if opposite_lte(a, b) and opposite_lte(b, c) {
        b <= a
        c <= b
        lte_trans(c, b, a)
        c <= a
        opposite_lte(a, c)
    }
}

/// On a partial order, mutual opposite bounds force equality.
theorem opposite_lte_antisymm[P: PartialOrder](a: P, b: P) {
    opposite_lte(a, b) and opposite_lte(b, a) implies a = b
} by {
    if opposite_lte(a, b) and opposite_lte(b, a) {
        b <= a
        a <= b
        lte_antisymm(a, b)
    }
}

/// On a linear order, the opposite relation is total.
theorem opposite_lte_total[L: LinearOrder](a: L, b: L) {
    opposite_lte(a, b) or opposite_lte(b, a)
} by {
    lte_or_gte(a, b)
    if a <= b {
        opposite_lte(b, a)
    }
    if a >= b {
        b <= a
        opposite_lte(a, b)
    }
}

/// The opposite relation agrees with `>=` on a partial order.
theorem opposite_lte_eq_gte[P: PartialOrder](a: P, b: P) {
    opposite_lte(a, b) = a >= b
} by {
    opposite_lte(a, b) = b <= a
    a >= b = b <= a
}
