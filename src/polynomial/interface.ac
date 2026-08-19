/// Public interface for univariate polynomials.

from algebra.add import Add
from finite_set import FiniteSet
from algebra.module.finite_support import has_support_in, is_finitely_supported
from algebra.field.field import Field
from data.basic.function_algebra import pointwise_add, pointwise_neg
from list import List, partial
from nat import Nat
from algebra.one import One
from algebra.ring.ring import Ring
from semiring import Semiring
from algebra.semiring_hom import SemiringHom
from comm_ring import CommRing
from data.basic.set import Set
from algebra.zero import Zero

numerals Nat

/// Univariate polynomials as finitely supported coefficient functions.

/// A univariate polynomial over `R`, represented by finitely supported coefficients.
structure Polynomial[R: Semiring] {
    /// The coefficient of `X^n`.
    coeff: Nat -> R
} constraint {
    is_finitely_supported(coeff)
}

/// Polynomial extensionality from equality of coefficient functions.
theorem polynomial_ext[R: Semiring](p: Polynomial[R], q: Polynomial[R]) {
    p.coeff = q.coeff implies p = q
}

/// Polynomial extensionality from pointwise equality of coefficients.
theorem polynomial_ext_pointwise[R: Semiring](p: Polynomial[R], q: Polynomial[R]) {
    (forall(n: Nat) { p.coeff(n) = q.coeff(n) }) implies p = q
}

/// Equal polynomials have equal coefficient functions.
theorem polynomial_eq_coeff[R: Semiring](p: Polynomial[R], q: Polynomial[R]) {
    p = q implies p.coeff = q.coeff
}

/// Equal polynomials have equal coefficients at every exponent.
theorem polynomial_eq_coeff_at[R: Semiring](p: Polynomial[R], q: Polynomial[R], n: Nat) {
    p = q implies p.coeff(n) = q.coeff(n)
}

/// The singleton support at one exponent.
let polynomial_singleton_support(n: Nat) -> result: FiniteSet[Nat] satisfy {
    FiniteSet[Nat].new(Set[Nat].singleton(n)) = Option.some(result)
}

/// The coefficient function of a monomial `r * X^n`.
define polynomial_monomial_coeff[R: Semiring](n: Nat, r: R, k: Nat) -> R {
    if k = n {
        r
    } else {
        R.0
    }
}

/// A monomial coefficient function is zero outside its singleton support.
theorem polynomial_monomial_coeff_zero_of_not_contains[R: Semiring](n: Nat, r: R, k: Nat) {
    not polynomial_singleton_support(n).contains(k) implies
    polynomial_monomial_coeff[R](n, r, k) = R.0
}

/// A monomial coefficient function is supported at its exponent.
theorem polynomial_monomial_coeff_has_support[R: Semiring](n: Nat, r: R) {
    has_support_in(polynomial_monomial_coeff[R](n, r), polynomial_singleton_support(n))
}

/// A monomial coefficient function is finitely supported.
theorem polynomial_monomial_coeff_is_finitely_supported[R: Semiring](n: Nat, r: R) {
    is_finitely_supported(polynomial_monomial_coeff[R](n, r))
}

/// The monomial polynomial `r * X^n`.
let polynomial_monomial[R: Semiring](n: Nat, r: R) -> result: Polynomial[R] satisfy {
    Polynomial[R].new(polynomial_monomial_coeff[R](n, r)) = Option.some(result)
}

/// The constant polynomial with coefficient `r`.
define polynomial_constant[R: Semiring](r: R) -> Polynomial[R] {
    polynomial_monomial(Nat.0, r)
}

/// The zero polynomial.
let polynomial_zero[R: Semiring]: Polynomial[R] = polynomial_constant(R.0)

/// The one polynomial.
let polynomial_one[R: Semiring]: Polynomial[R] = polynomial_constant(R.1)

/// The coefficientwise sum of two polynomials.
let polynomial_add[R: Semiring](p: Polynomial[R], q: Polynomial[R]) -> result: Polynomial[R] satisfy {
    Polynomial[R].new(pointwise_add(p.coeff, q.coeff)) = Option.some(result)
}

attributes Polynomial[R: Semiring] {
    /// Polynomial extensionality from equality of coefficients.
    let ext = polynomial_ext[R]

    /// The monomial `r * X^n`.
    let monomial: (Nat, R) -> Polynomial[R] = polynomial_monomial

    /// The constant polynomial with coefficient `r`.
    let constant: R -> Polynomial[R] = polynomial_constant

    /// The zero polynomial.
    let zero: Polynomial[R] = polynomial_zero[R]

    /// The one polynomial.
    let one: Polynomial[R] = polynomial_one[R]

    /// Coefficientwise addition of polynomials.
    define add(self, other: Polynomial[R]) -> Polynomial[R] {
        polynomial_add(self, other)
    }
}

/// Polynomials have coefficientwise addition.
instance Polynomial[R: Semiring]: Add {
    let add = Polynomial[R].add
}

/// Polynomials have a zero element.
instance Polynomial[R: Semiring]: Zero {
    let 0 = Polynomial[R].zero
}

/// Polynomials have the constant-one element.
instance Polynomial[R: Semiring]: One {
    let 1 = Polynomial[R].one
}

/// The zero polynomial has zero coefficient at every exponent.
theorem polynomial_zero_coeff[R: Semiring](n: Nat) {
    Polynomial[R].zero.coeff(n) = R.0
}

/// Addition of polynomials adds coefficients pointwise.
theorem polynomial_add_coeff[R: Semiring](p: Polynomial[R], q: Polynomial[R], n: Nat) {
    (p + q).coeff(n) = p.coeff(n) + q.coeff(n)
}

/// The monomial has the chosen coefficient at its exponent.
theorem polynomial_monomial_coeff_self[R: Semiring](n: Nat, r: R) {
    Polynomial[R].monomial(n, r).coeff(n) = r
}

/// The monomial is zero away from its exponent.
theorem polynomial_monomial_coeff_of_ne[R: Semiring](n: Nat, r: R, k: Nat) {
    k != n implies Polynomial[R].monomial(n, r).coeff(k) = R.0
}

/// A constant polynomial has its constant coefficient.
theorem polynomial_constant_coeff_zero[R: Semiring](r: R) {
    Polynomial[R].constant(r).coeff(Nat.0) = r
}

/// A constant polynomial is zero at every nonzero exponent.
theorem polynomial_constant_coeff_of_ne_zero[R: Semiring](r: R, n: Nat) {
    n != Nat.0 implies Polynomial[R].constant(r).coeff(n) = R.0
}

/// The one polynomial has constant coefficient one.
theorem polynomial_one_coeff_zero[R: Semiring] {
    Polynomial[R].one.coeff(Nat.0) = R.1
}

/// The one polynomial is zero at every nonzero exponent.
theorem polynomial_one_coeff_of_ne_zero[R: Semiring](n: Nat) {
    n != Nat.0 implies Polynomial[R].one.coeff(n) = R.0
}

/// The zero polynomial is supported in every finite set.
theorem polynomial_zero_has_support_in[R: Semiring](s: FiniteSet[Nat]) {
    has_support_in(Polynomial[R].zero.coeff, s)
}

/// The zero polynomial is supported in the empty finite set.
theorem polynomial_zero_has_support_empty[R: Semiring] {
    has_support_in(Polynomial[R].zero.coeff, FiniteSet[Nat].empty)
}

/// A polynomial supported in the empty finite set is the zero polynomial.
theorem polynomial_eq_zero_of_has_support_empty[R: Semiring](p: Polynomial[R]) {
    has_support_in(p.coeff, FiniteSet[Nat].empty) implies p = Polynomial[R].zero
}

/// A monomial is supported in its singleton support.
theorem polynomial_monomial_has_support_in_singleton[R: Semiring](n: Nat, r: R) {
    has_support_in(Polynomial[R].monomial(n, r).coeff, polynomial_singleton_support(n))
}

/// The sum of two polynomials is supported in the union of supports.
theorem polynomial_add_has_support_in_union[R: Semiring](
    p: Polynomial[R],
    q: Polynomial[R],
    s: FiniteSet[Nat],
    u: FiniteSet[Nat]
) {
    has_support_in(p.coeff, s) and has_support_in(q.coeff, u) implies
    has_support_in(polynomial_add(p, q).coeff, s.union(u))
}

/// Addition of polynomials is associative.
theorem polynomial_add_assoc[R: Semiring](p: Polynomial[R], q: Polynomial[R], r: Polynomial[R]) {
    polynomial_add(p, polynomial_add(q, r)) = polynomial_add(polynomial_add(p, q), r)
}

/// Addition of polynomials is commutative.
theorem polynomial_add_comm[R: Semiring](p: Polynomial[R], q: Polynomial[R]) {
    polynomial_add(p, q) = polynomial_add(q, p)
}

/// Adding zero on the right changes no polynomial.
theorem polynomial_add_zero_right[R: Semiring](p: Polynomial[R]) {
    polynomial_add(p, polynomial_zero[R]) = p
}

/// Adding zero on the left changes no polynomial.
theorem polynomial_add_zero_left[R: Semiring](p: Polynomial[R]) {
    polynomial_add(polynomial_zero[R], p) = p
}

/// A polynomial is zero when all its coefficients are zero.
theorem polynomial_eq_zero_of_coeff_zero[R: Semiring](p: Polynomial[R]) {
    (forall(n: Nat) { p.coeff(n) = R.0 }) implies p = Polynomial[R].zero
}

/// A zero polynomial has zero coefficients.
theorem polynomial_coeff_zero_of_eq_zero[R: Semiring](p: Polynomial[R], n: Nat) {
    p = Polynomial[R].zero implies p.coeff(n) = R.0
}

/// The coefficientwise additive inverse of a polynomial.
let polynomial_neg[R: Ring](p: Polynomial[R]) -> result: Polynomial[R] satisfy {
    Polynomial[R].new(pointwise_neg(p.coeff)) = Option.some(result)
}

attributes Polynomial[R: Ring] {
    /// The coefficientwise additive inverse of a polynomial.
    define neg(self) -> Polynomial[R] {
        polynomial_neg(self)
    }

    /// The coefficientwise difference of two polynomials.
    define sub(self, other: Polynomial[R]) -> Polynomial[R] {
        self + other.neg
    }
}

/// Negation of a polynomial negates every coefficient.
theorem polynomial_neg_coeff[R: Ring](p: Polynomial[R], n: Nat) {
    p.neg.coeff(n) = -p.coeff(n)
}

/// The coefficientwise difference subtracts coefficients.
theorem polynomial_sub_coeff[R: Ring](p: Polynomial[R], q: Polynomial[R], n: Nat) {
    p.sub(q).coeff(n) = p.coeff(n) + -q.coeff(n)
}

/// Negation preserves a specified support.
theorem polynomial_neg_has_support_in[R: Ring](p: Polynomial[R], s: FiniteSet[Nat]) {
    has_support_in(p.coeff, s) implies has_support_in(p.neg.coeff, s)
}

/// Subtraction is supported in the union of supports.
theorem polynomial_sub_has_support_in_union[R: Ring](
    p: Polynomial[R],
    q: Polynomial[R],
    s: FiniteSet[Nat],
    u: FiniteSet[Nat]
) {
    has_support_in(p.coeff, s) and has_support_in(q.coeff, u) implies
    has_support_in(p.sub(q).coeff, s.union(u))
}

/// Adding the additive inverse on the right gives zero.
theorem polynomial_add_neg_right[R: Ring](p: Polynomial[R]) {
    polynomial_add(p, p.neg) = Polynomial[R].zero
}

/// Subtracting a polynomial from itself gives zero.
theorem polynomial_sub_self[R: Ring](p: Polynomial[R]) {
    p.sub(p) = Polynomial[R].zero
}

/// Bounded coefficient-polynomial evaluation and linear-factor theorems.

/// The coefficient function obtained by dropping the constant coefficient.
define coeff_tail[R](c: Nat -> R, i: Nat) -> R {
    c(i.suc)
}

/// Horner evaluation of the polynomial determined by the first `n` coefficients.
define coeff_eval[R: Semiring](c: Nat -> R, x: R, n: Nat) -> R {
    match n {
        Nat.zero {
            R.0
        }
        Nat.suc(pred) {
            c(Nat.0) + x * coeff_eval(coeff_tail(c), x, pred)
        }
    }
}

/// The synthetic quotient coefficients for division by `X - a`, using the first `n` coefficients.
define coeff_quotient[R: Semiring](c: Nat -> R, a: R, n: Nat, i: Nat) -> R {
    match n {
        Nat.zero {
            R.0
        }
        Nat.suc(pred) {
            match i {
                Nat.zero {
                    coeff_eval(coeff_tail(c), a, pred)
                }
                Nat.suc(ipred) {
                    coeff_quotient(coeff_tail(c), a, pred, ipred)
                }
            }
        }
    }
}

/// The value at `x` of a quotient polynomial multiplied by the linear factor `X - a`.
define coeff_mul_x_minus_a[R: CommRing](q: Nat -> R, a: R, x: R, n: Nat) -> R {
    coeff_eval(q, x, n) * (x - a)
}

/// The assertion that a coefficient function vanishes at one index.
define coeff_zero_at[R: Semiring](c: Nat -> R, k: Nat) -> Bool {
    c(k) = R.0
}

/// The implication that one coefficient outside a bound is zero.
define coeff_zero_from_at[R: Semiring](c: Nat -> R, n: Nat, k: Nat) -> Bool {
    not k < n implies coeff_zero_at(c, k)
}

/// A coefficient function is zero from the bound onward.
define coeff_zero_from[R: Semiring](c: Nat -> R, n: Nat) -> Bool {
    forall(k: Nat) {
        coeff_zero_from_at(c, n, k)
    }
}

/// The assertion that a finite set has no exponent outside a strict bound at one index.
define finite_set_nat_outside_bound(s: FiniteSet[Nat], n: Nat, k: Nat) -> Bool {
    not k < n implies not s.contains(k)
}

/// A finite set of natural-number exponents is strictly bounded by `n`.
define finite_set_nat_bounded_by(s: FiniteSet[Nat], n: Nat) -> Bool {
    forall(k: Nat) {
        finite_set_nat_outside_bound(s, n, k)
    }
}

/// Applying a zero-from-bound witness at an index outside the bound gives a zero coefficient.
theorem coeff_zero_from_apply[R: Semiring](c: Nat -> R, n: Nat, k: Nat) {
    coeff_zero_from(c, n) and not k < n implies c(k) = R.0
}

/// A finite-set bound excludes any exponent outside the bound.
theorem finite_set_nat_bounded_by_not_contains(s: FiniteSet[Nat], n: Nat, k: Nat) {
    finite_set_nat_bounded_by(s, n) and not k < n implies not s.contains(k)
}

/// Applying a finite-set exponent bound at a member gives the strict bound.
theorem finite_set_nat_bounded_by_apply(s: FiniteSet[Nat], n: Nat, k: Nat) {
    finite_set_nat_bounded_by(s, n) and s.contains(k) implies k < n
}

define coeff_eval_add_at[R: Semiring](c: Nat -> R, d: Nat -> R, x: R, n: Nat) -> Bool {
    coeff_eval(pointwise_add(c, d), x, n) = coeff_eval(c, x, n) + coeff_eval(d, x, n)
}

define coeff_eval_add_property[R: Semiring](x: R, n: Nat) -> Bool {
    forall(c: Nat -> R, d: Nat -> R) {
        coeff_eval_add_at(c, d, x, n)
    }
}

/// Evaluating a coefficient sequence at bound zero yields the zero element.
theorem coeff_eval_zero_index[R: Semiring](c: Nat -> R, x: R) {
    coeff_eval(c, x, Nat.0) = R.0
}

theorem coeff_eval_add[R: Semiring](c: Nat -> R, d: Nat -> R, x: R, n: Nat) {
    coeff_eval(pointwise_add(c, d), x, n) = coeff_eval(c, x, n) + coeff_eval(d, x, n)
}

/// A bounded coefficient polynomial is divisible by `X - a` when it is represented by such a product.
define coeff_divisible_by_x_minus_a[R: CommRing](c: Nat -> R, a: R, n: Nat) -> Bool {
    exists(q: Nat -> R) {
        forall(x: R) {
            coeff_eval(c, x, n) = coeff_mul_x_minus_a(q, a, x, n)
        }
    }
}

/// The constant coefficient of the synthetic quotient is the tail evaluated at the root.
theorem coeff_quotient_zero_index[R: Semiring](c: Nat -> R, a: R, n: Nat) {
    coeff_quotient(c, a, n.suc, Nat.0) = coeff_eval(coeff_tail(c), a, n)
}

/// The tail of the synthetic quotient is the synthetic quotient of the tail.
theorem coeff_tail_quotient[R: Semiring](c: Nat -> R, a: R, n: Nat) {
    coeff_tail(coeff_quotient(c, a, n.suc), Nat.0) = coeff_quotient(coeff_tail(c), a, n, Nat.0) and
    coeff_tail(coeff_quotient(c, a, n.suc)) = coeff_quotient(coeff_tail(c), a, n)
}

/// The bounded remainder identity for a fixed coefficient function.
define coeff_remainder_at[R: CommRing](c: Nat -> R, a: R, x: R, n: Nat) -> Bool {
    coeff_eval(c, x, n) = coeff_eval(c, a, n) + (x - a) * coeff_eval(coeff_quotient(c, a, n), x, n)
}

/// The bounded remainder identity for all coefficient functions at a fixed bound.
define coeff_remainder_property[R: CommRing](a: R, x: R, n: Nat) -> Bool {
    forall(c: Nat -> R) {
        coeff_remainder_at(c, a, x, n)
    }
}

/// The bounded coefficient-form remainder theorem.
theorem coeff_remainder_theorem[R: CommRing](c: Nat -> R, a: R, x: R, n: Nat) {
    coeff_eval(c, x, n) = coeff_eval(c, a, n) + (x - a) * coeff_eval(coeff_quotient(c, a, n), x, n)
}

/// A root gives divisibility by the corresponding linear factor.
theorem coeff_factor_theorem_forward[R: CommRing](c: Nat -> R, a: R, n: Nat) {
    coeff_eval(c, a, n) = R.0 implies coeff_divisible_by_x_minus_a(c, a, n)
}

/// Divisibility by `X - a` forces `a` to be a root.
theorem coeff_factor_theorem_reverse[R: CommRing](c: Nat -> R, a: R, n: Nat) {
    coeff_divisible_by_x_minus_a(c, a, n) implies coeff_eval(c, a, n) = R.0
}

/// The bounded coefficient-form factor theorem for commutative rings.
theorem coeff_factor_theorem[R: CommRing](c: Nat -> R, a: R, n: Nat) {
    coeff_eval(c, a, n) = R.0 = coeff_divisible_by_x_minus_a(c, a, n)
}

/// A polynomial has no nonzero coefficient at or above `n`.
define polynomial_support_bounded_by[R: Semiring](p: Polynomial[R], n: Nat) -> Bool {
    coeff_zero_from(p.coeff, n)
}

/// Bounded Horner evaluation of a polynomial using its first `n` coefficients.
define polynomial_eval_bound[R: Semiring](p: Polynomial[R], x: R, n: Nat) -> R {
    coeff_eval(p.coeff, x, n)
}

/// Bounded polynomial evaluation unfolds to coefficient evaluation on the coefficient function.
theorem polynomial_eval_bound_eq_coeff_eval[R: Semiring](p: Polynomial[R], x: R, n: Nat) {
    polynomial_eval_bound(p, x, n) = coeff_eval(p.coeff, x, n)
}

/// The synthetic quotient coefficients for division by `X - a`, at a fixed bound.
define polynomial_quotient_coeff[R: Semiring](p: Polynomial[R], a: R, n: Nat, i: Nat) -> R {
    coeff_quotient(p.coeff, a, n, i)
}

/// Bounded evaluation of the synthetic quotient associated to a polynomial.
define polynomial_quotient_eval_bound[R: Semiring](p: Polynomial[R], a: R, x: R, n: Nat) -> R {
    coeff_eval(polynomial_quotient_coeff(p, a, n), x, n)
}

/// A synthetic quotient coefficient function is finitely supported.
theorem polynomial_quotient_coeff_is_finitely_supported[R: Semiring](p: Polynomial[R], a: R, n: Nat) {
    is_finitely_supported(polynomial_quotient_coeff(p, a, n))
}

/// The synthetic quotient polynomial for division by `X - a`, at a fixed bound.
let polynomial_quotient[R: Semiring](p: Polynomial[R], a: R, n: Nat) -> result: Polynomial[R] satisfy {
    Polynomial[R].new(polynomial_quotient_coeff(p, a, n)) = Option.some(result)
}

/// The bundled synthetic quotient has the synthetic quotient coefficients.
theorem polynomial_quotient_coeff_eq[R: Semiring](p: Polynomial[R], a: R, n: Nat) {
    polynomial_quotient(p, a, n).coeff = polynomial_quotient_coeff(p, a, n)
}

/// One coefficient of the bundled synthetic quotient is the corresponding synthetic quotient coefficient.
theorem polynomial_quotient_coeff_at[R: Semiring](p: Polynomial[R], a: R, n: Nat, k: Nat) {
    polynomial_quotient(p, a, n).coeff(k) = polynomial_quotient_coeff(p, a, n, k)
}

/// Bounded evaluation of the bundled synthetic quotient matches quotient evaluation.
theorem polynomial_quotient_eval_bound_eq[R: Semiring](p: Polynomial[R], a: R, x: R, n: Nat) {
    polynomial_eval_bound(polynomial_quotient(p, a, n), x, n) = polynomial_quotient_eval_bound(p, a, x, n)
}

/// A polynomial witnesses bounded divisibility by the linear factor `X - a`.
define polynomial_x_minus_a_polynomial_witness_bound[R: CommRing](
    p: Polynomial[R],
    a: R,
    n: Nat,
    q: Polynomial[R]
) -> Bool {
    forall(x: R) {
        polynomial_eval_bound(p, x, n) = polynomial_eval_bound(q, x, n) * (x - a)
    }
}

/// Applying a polynomial quotient witness at one evaluation point.
theorem polynomial_x_minus_a_polynomial_witness_bound_apply[R: CommRing](
    p: Polynomial[R],
    a: R,
    n: Nat,
    q: Polynomial[R],
    x: R
) {
    polynomial_x_minus_a_polynomial_witness_bound(p, a, n, q) implies
    polynomial_eval_bound(p, x, n) = polynomial_eval_bound(q, x, n) * (x - a)
}

/// Bounded divisibility by `X - a` with a polynomial quotient witness.
define polynomial_divisible_by_x_minus_a_polynomial_bound[R: CommRing](p: Polynomial[R], a: R, n: Nat) -> Bool {
    exists(q: Polynomial[R]) {
        polynomial_x_minus_a_polynomial_witness_bound(p, a, n, q)
    }
}

/// A coefficient function witnesses bounded divisibility by the linear factor `X - a`.
define polynomial_x_minus_a_witness_bound[R: CommRing](
    p: Polynomial[R],
    a: R,
    n: Nat,
    q: Nat -> R
) -> Bool {
    forall(x: R) {
        polynomial_eval_bound(p, x, n) = coeff_mul_x_minus_a(q, a, x, n)
    }
}

/// Bounded divisibility by the linear factor `X - a` at a fixed evaluation bound.
define polynomial_divisible_by_x_minus_a_bound[R: CommRing](p: Polynomial[R], a: R, n: Nat) -> Bool {
    exists(q: Nat -> R) {
        polynomial_x_minus_a_witness_bound(p, a, n, q)
    }
}

/// A finite support contained below `n` gives a polynomial support bound at `n`.
theorem polynomial_support_bounded_by_of_has_support_in[R: Semiring](
    p: Polynomial[R],
    s: FiniteSet[Nat],
    n: Nat
) {
    has_support_in(p.coeff, s) and finite_set_nat_bounded_by(s, n) implies
    polynomial_support_bounded_by(p, n)
}

/// The bundled synthetic quotient has support bounded by its explicit bound.
theorem polynomial_quotient_support_bounded_by[R: Semiring](p: Polynomial[R], a: R, n: Nat) {
    polynomial_support_bounded_by(polynomial_quotient(p, a, n), n)
}

/// Bounded polynomial evaluation is independent of increasing a valid support bound.
theorem polynomial_eval_bound_eq_of_support_bounded_le[R: Semiring](
    p: Polynomial[R],
    x: R,
    n: Nat,
    m: Nat
) {
    polynomial_support_bounded_by(p, n) and n <= m implies
    polynomial_eval_bound(p, x, m) = polynomial_eval_bound(p, x, n)
}

/// A concrete finite support bound gives independence after enlarging the evaluation bound.
theorem polynomial_eval_bound_eq_of_has_support_in_le[R: Semiring](
    p: Polynomial[R],
    s: FiniteSet[Nat],
    x: R,
    n: Nat,
    m: Nat
) {
    has_support_in(p.coeff, s) and finite_set_nat_bounded_by(s, n) and n <= m implies
    polynomial_eval_bound(p, x, m) = polynomial_eval_bound(p, x, n)
}

/// Applying a polynomial support bound at an index outside the bound gives a zero coefficient.
theorem polynomial_support_bounded_by_apply[R: Semiring](p: Polynomial[R], n: Nat, k: Nat) {
    polynomial_support_bounded_by(p, n) and not k < n implies p.coeff(k) = R.0
}

/// Enlarging a polynomial support bound preserves boundedness.
theorem polynomial_support_bounded_by_monotone[R: Semiring](
    p: Polynomial[R],
    n: Nat,
    m: Nat
) {
    polynomial_support_bounded_by(p, n) and n <= m implies polynomial_support_bounded_by(p, m)
}

/// Bounded polynomial evaluation is independent of any chosen valid support bound.
theorem polynomial_eval_bound_eq_of_support_bounded[R: Semiring](
    p: Polynomial[R],
    x: R,
    n: Nat,
    m: Nat
) {
    polynomial_support_bounded_by(p, n) and polynomial_support_bounded_by(p, m) implies
    polynomial_eval_bound(p, x, n) = polynomial_eval_bound(p, x, m)
}

/// A sum is support-bounded when both summands are bounded by the same exponent.
theorem polynomial_add_support_bounded_by[R: Semiring](p: Polynomial[R], q: Polynomial[R], n: Nat) {
    polynomial_support_bounded_by(p, n) and polynomial_support_bounded_by(q, n) implies
    polynomial_support_bounded_by(p + q, n)
}

/// The zero polynomial has support bounded by zero.
theorem polynomial_zero_support_bounded_by_zero[R: Semiring] {
    polynomial_support_bounded_by(Polynomial[R].zero, Nat.0)
}

/// Every bounded evaluation of the zero polynomial is zero.
theorem polynomial_eval_bound_zero[R: Semiring](x: R, n: Nat) {
    polynomial_eval_bound(Polynomial[R].zero, x, n) = R.0
}

/// The constant polynomial has support bounded by one coefficient slot.
theorem polynomial_constant_support_bounded_by_one[R: Semiring](r: R) {
    polynomial_support_bounded_by(Polynomial[R].constant(r), Nat.0.suc)
}

/// The one polynomial has support bounded by one coefficient slot.
theorem polynomial_one_support_bounded_by_one[R: Semiring] {
    polynomial_support_bounded_by(Polynomial[R].one, Nat.0.suc)
}

/// Bounded evaluation is additive at the same bound.
theorem polynomial_eval_bound_add[R: Semiring](p: Polynomial[R], q: Polynomial[R], x: R, n: Nat) {
    polynomial_eval_bound(p + q, x, n) = polynomial_eval_bound(p, x, n) + polynomial_eval_bound(q, x, n)
}

/// Evaluating a constant polynomial with one coefficient slot returns the constant.
theorem polynomial_eval_bound_constant_one[R: Semiring](r: R, x: R) {
    polynomial_eval_bound(Polynomial[R].constant(r), x, Nat.0.suc) = r
}

/// Evaluating the one polynomial with one coefficient slot returns one.
theorem polynomial_eval_bound_one_one[R: Semiring](x: R) {
    polynomial_eval_bound(Polynomial[R].one, x, Nat.0.suc) = R.1
}

/// Every larger bounded evaluation of a constant polynomial returns the constant.
theorem polynomial_eval_bound_constant_of_one_le[R: Semiring](r: R, x: R, n: Nat) {
    Nat.0.suc <= n implies polynomial_eval_bound(Polynomial[R].constant(r), x, n) = r
}

/// Every larger bounded evaluation of the one polynomial returns one.
theorem polynomial_eval_bound_one_of_one_le[R: Semiring](x: R, n: Nat) {
    Nat.0.suc <= n implies polynomial_eval_bound(Polynomial[R].one, x, n) = R.1
}

/// Every finite set of natural-number exponents has a strict natural bound.
theorem finite_set_nat_bounded_by_exists(s: FiniteSet[Nat]) {
    exists(n: Nat) {
        finite_set_nat_bounded_by(s, n)
    }
}

/// Every polynomial has some strict coefficient-support bound.
theorem polynomial_support_bound_exists[R: Semiring](p: Polynomial[R]) {
    exists(n: Nat) {
        polynomial_support_bounded_by(p, n)
    }
}

/// Support-independent polynomial evaluation.
let polynomial_eval[R: Semiring](p: Polynomial[R], x: R) -> result: R satisfy {
    forall(n: Nat) {
        polynomial_support_bounded_by(p, n) implies result = polynomial_eval_bound(p, x, n)
    }
}

/// Global evaluation agrees with bounded evaluation at any valid support bound.
theorem polynomial_eval_eq_eval_bound_of_support_bounded[R: Semiring](p: Polynomial[R], x: R, n: Nat) {
    polynomial_support_bounded_by(p, n) implies polynomial_eval(p, x) = polynomial_eval_bound(p, x, n)
}

/// Global evaluation of the zero polynomial is zero.
theorem polynomial_eval_zero[R: Semiring](x: R) {
    polynomial_eval(Polynomial[R].zero, x) = R.0
}

/// Global evaluation of a constant polynomial is its constant value.
theorem polynomial_eval_constant[R: Semiring](r: R, x: R) {
    polynomial_eval(Polynomial[R].constant(r), x) = r
}

/// Global evaluation of the one polynomial is one.
theorem polynomial_eval_one[R: Semiring](x: R) {
    polynomial_eval(Polynomial[R].one, x) = R.1
}

/// Global evaluation is additive.
theorem polynomial_eval_add[R: Semiring](p: Polynomial[R], q: Polynomial[R], x: R) {
    polynomial_eval(p + q, x) = polynomial_eval(p, x) + polynomial_eval(q, x)
}

/// Global evaluation sends coefficientwise negation to additive inverse.
theorem polynomial_eval_neg[R: Ring](p: Polynomial[R], x: R) {
    polynomial_eval(p.neg, x) = -polynomial_eval(p, x)
}

/// Global evaluation sends coefficientwise subtraction to subtraction of values.
theorem polynomial_eval_sub[R: Ring](p: Polynomial[R], q: Polynomial[R], x: R) {
    polynomial_eval(p.sub(q), x) = polynomial_eval(p, x) - polynomial_eval(q, x)
}

/// The pointwise statement that a polynomial quotient witnesses divisibility by `X - a`.
define polynomial_x_minus_a_polynomial_witness_at[R: CommRing](
    p: Polynomial[R],
    a: R,
    q: Polynomial[R],
    x: R
) -> Bool {
    polynomial_eval(p, x) = polynomial_eval(q, x) * (x - a)
}

/// A polynomial quotient witnesses global divisibility by the linear factor `X - a`.
define polynomial_x_minus_a_polynomial_witness[R: CommRing](
    p: Polynomial[R],
    a: R,
    q: Polynomial[R]
) -> Bool {
    forall(x: R) {
        polynomial_x_minus_a_polynomial_witness_at(p, a, q, x)
    }
}

/// Global polynomial-witness divisibility by the linear factor `X - a`.
define polynomial_divisible_by_x_minus_a_polynomial[R: CommRing](p: Polynomial[R], a: R) -> Bool {
    exists(q: Polynomial[R]) {
        polynomial_x_minus_a_polynomial_witness(p, a, q)
    }
}

/// Applying a global polynomial quotient witness at one evaluation point.
theorem polynomial_x_minus_a_polynomial_witness_apply[R: CommRing](
    p: Polynomial[R],
    a: R,
    q: Polynomial[R],
    x: R
) {
    polynomial_x_minus_a_polynomial_witness(p, a, q) implies
    polynomial_eval(p, x) = polynomial_eval(q, x) * (x - a)
}

/// At a valid support bound, the bundled synthetic quotient is a global witness for a root.
theorem polynomial_quotient_global_witness_of_support_bounded[R: CommRing](p: Polynomial[R], a: R, n: Nat) {
    polynomial_support_bounded_by(p, n) and polynomial_eval(p, a) = R.0 implies
    polynomial_x_minus_a_polynomial_witness(p, a, polynomial_quotient(p, a, n))
}

/// At any larger valid support bound, the bundled synthetic quotient is a global witness for a root.
theorem polynomial_quotient_global_witness_of_support_bounded_le[R: CommRing](
    p: Polynomial[R],
    a: R,
    n: Nat,
    m: Nat
) {
    polynomial_support_bounded_by(p, n) and n <= m and polynomial_eval(p, a) = R.0 implies
    polynomial_x_minus_a_polynomial_witness(p, a, polynomial_quotient(p, a, m))
}

/// A global root gives global divisibility by `X - a` with a polynomial witness.
theorem polynomial_factor_theorem_forward_global[R: CommRing](p: Polynomial[R], a: R) {
    polynomial_eval(p, a) = R.0 implies polynomial_divisible_by_x_minus_a_polynomial(p, a)
}

/// Global divisibility by `X - a` forces `a` to be a global root.
theorem polynomial_factor_theorem_reverse_global[R: CommRing](p: Polynomial[R], a: R) {
    polynomial_divisible_by_x_minus_a_polynomial(p, a) implies polynomial_eval(p, a) = R.0
}

/// The global polynomial factor theorem for the polynomial-witness divisibility predicate.
theorem polynomial_factor_theorem_global[R: CommRing](p: Polynomial[R], a: R) {
    polynomial_eval(p, a) = R.0 = polynomial_divisible_by_x_minus_a_polynomial(p, a)
}

/// The `i`th summand in the coefficient of `X^k` in a product.
define polynomial_mul_term_coeff[R: Semiring](p: Polynomial[R], q: Polynomial[R], k: Nat, i: Nat) -> R {
    p.coeff(i) * q.coeff(k - i)
}

/// The coefficient of `X^k` in the convolution product.
define polynomial_mul_coeff[R: Semiring](p: Polynomial[R], q: Polynomial[R], k: Nat) -> R {
    partial(function(i: Nat) { polynomial_mul_term_coeff(p, q, k, i) }, k.suc)
}

/// If both factors are support-bounded and `k` is outside the sum bound, the convolution coefficient vanishes.
theorem polynomial_mul_coeff_zero_of_support_bounded[R: Semiring](
    p: Polynomial[R],
    q: Polynomial[R],
    m: Nat,
    n: Nat,
    k: Nat
) {
    polynomial_support_bounded_by(p, m) and polynomial_support_bounded_by(q, n) and
    not k < m + n implies polynomial_mul_coeff(p, q, k) = R.0
}

/// The convolution coefficient function is finitely supported.
theorem polynomial_mul_coeff_is_finitely_supported[R: Semiring](p: Polynomial[R], q: Polynomial[R]) {
    is_finitely_supported(polynomial_mul_coeff(p, q))
}

/// The bundled polynomial product with convolution coefficients.
let polynomial_mul[R: Semiring](p: Polynomial[R], q: Polynomial[R]) -> result: Polynomial[R] satisfy {
    Polynomial[R].new(polynomial_mul_coeff(p, q)) = Option.some(result)
}

/// The bundled product has the convolution coefficient function.
theorem polynomial_mul_coeff_eq[R: Semiring](p: Polynomial[R], q: Polynomial[R]) {
    polynomial_mul(p, q).coeff = polynomial_mul_coeff(p, q)
}

/// The `k`th coefficient of the bundled product is the convolution coefficient.
theorem polynomial_mul_coeff_apply[R: Semiring](p: Polynomial[R], q: Polynomial[R], k: Nat) {
    polynomial_mul(p, q).coeff(k) = polynomial_mul_coeff(p, q, k)
}

/// The product of polynomials supported below `m` and `n` is supported below `m + n`.
theorem polynomial_mul_support_bounded_by_add[R: Semiring](
    p: Polynomial[R],
    q: Polynomial[R],
    m: Nat,
    n: Nat
) {
    polynomial_support_bounded_by(p, m) and polynomial_support_bounded_by(q, n) implies
    polynomial_support_bounded_by(polynomial_mul(p, q), m + n)
}

/// Bounded evaluation of a polynomial product is the product of bounded evaluations.
theorem polynomial_eval_bound_mul_of_support_bounded[R: CommRing](
    p: Polynomial[R],
    q: Polynomial[R],
    x: R,
    m: Nat,
    n: Nat
) {
    polynomial_support_bounded_by(p, m) and polynomial_support_bounded_by(q, n) implies
    polynomial_eval_bound(polynomial_mul(p, q), x, m + n) =
        polynomial_eval_bound(p, x, m) * polynomial_eval_bound(q, x, n)
}

/// Global evaluation of a polynomial product is the product of global evaluations.
theorem polynomial_eval_mul[R: CommRing](p: Polynomial[R], q: Polynomial[R], x: R) {
    polynomial_eval(polynomial_mul(p, q), x) = polynomial_eval(p, x) * polynomial_eval(q, x)
}

/// Evaluation at a fixed point, viewed as a function from polynomials to the coefficient ring.
define polynomial_eval_at[R: Semiring](x: R, p: Polynomial[R]) -> R {
    polynomial_eval(p, x)
}

/// Evaluation at a fixed point sends the zero polynomial to zero.
theorem polynomial_eval_at_zero[R: Semiring](x: R) {
    polynomial_eval_at(x, Polynomial[R].zero) = R.0
}

/// Evaluation at a fixed point sends a constant polynomial to the represented coefficient.
theorem polynomial_eval_at_constant[R: Semiring](x: R, r: R) {
    polynomial_eval_at(x, Polynomial[R].constant(r)) = r
}

/// Evaluation at a fixed point sends the one polynomial to one.
theorem polynomial_eval_at_one[R: Semiring](x: R) {
    polynomial_eval_at(x, Polynomial[R].one) = R.1
}

/// Evaluation at a fixed point preserves polynomial addition.
theorem polynomial_eval_at_add[R: Semiring](x: R, p: Polynomial[R], q: Polynomial[R]) {
    polynomial_eval_at(x, p + q) = polynomial_eval_at(x, p) + polynomial_eval_at(x, q)
}

/// Evaluation at a fixed point preserves the polynomial product.
theorem polynomial_eval_at_mul[R: CommRing](x: R, p: Polynomial[R], q: Polynomial[R]) {
    polynomial_eval_at(x, polynomial_mul(p, q)) = polynomial_eval_at(x, p) * polynomial_eval_at(x, q)
}

/// Evaluation at a fixed point preserves coefficientwise additive inverse.
theorem polynomial_eval_at_neg[R: Ring](x: R, p: Polynomial[R]) {
    polynomial_eval_at(x, p.neg) = -polynomial_eval_at(x, p)
}

/// Evaluation at a fixed point preserves coefficientwise subtraction.
theorem polynomial_eval_at_sub[R: Ring](x: R, p: Polynomial[R], q: Polynomial[R]) {
    polynomial_eval_at(x, p.sub(q)) = polynomial_eval_at(x, p) - polynomial_eval_at(x, q)
}

/// The bounded remainder identity for a polynomial and explicit evaluation bound.
theorem polynomial_remainder_theorem_bound[R: CommRing](p: Polynomial[R], a: R, x: R, n: Nat) {
    polynomial_eval_bound(p, x, n) =
        polynomial_eval_bound(p, a, n) +
        (x - a) * polynomial_quotient_eval_bound(p, a, x, n)
}

/// The bounded remainder identity with the bundled synthetic quotient.
theorem polynomial_remainder_theorem_bundled_bound[R: CommRing](p: Polynomial[R], a: R, x: R, n: Nat) {
    polynomial_eval_bound(p, x, n) =
        polynomial_eval_bound(p, a, n) +
        (x - a) * polynomial_eval_bound(polynomial_quotient(p, a, n), x, n)
}

/// The bundled synthetic quotient witnesses divisibility at a zero bounded value.
theorem polynomial_factor_theorem_forward_bundled_bound[R: CommRing](p: Polynomial[R], a: R, n: Nat) {
    polynomial_eval_bound(p, a, n) = R.0 implies polynomial_divisible_by_x_minus_a_polynomial_bound(p, a, n)
}

/// Polynomial-witness bounded divisibility by `X - a` forces the bounded value at `a` to vanish.
theorem polynomial_factor_theorem_reverse_bundled_bound[R: CommRing](p: Polynomial[R], a: R, n: Nat) {
    polynomial_divisible_by_x_minus_a_polynomial_bound(p, a, n) implies polynomial_eval_bound(p, a, n) = R.0
}

/// The bundled bounded polynomial factor theorem for an explicit evaluation bound.
theorem polynomial_factor_theorem_bundled_bound[R: CommRing](p: Polynomial[R], a: R, n: Nat) {
    polynomial_eval_bound(p, a, n) = R.0 = polynomial_divisible_by_x_minus_a_polynomial_bound(p, a, n)
}

/// A zero bounded value gives bounded divisibility by `X - a` at the same explicit bound.
theorem polynomial_factor_theorem_forward_bound[R: CommRing](p: Polynomial[R], a: R, n: Nat) {
    polynomial_eval_bound(p, a, n) = R.0 implies polynomial_divisible_by_x_minus_a_bound(p, a, n)
}

/// Bounded divisibility by `X - a` forces the bounded evaluation at `a` to vanish.
theorem polynomial_factor_theorem_reverse_bound[R: CommRing](p: Polynomial[R], a: R, n: Nat) {
    polynomial_divisible_by_x_minus_a_bound(p, a, n) implies polynomial_eval_bound(p, a, n) = R.0
}

/// The bounded polynomial factor theorem for an explicit evaluation bound.
theorem polynomial_factor_theorem_bound[R: CommRing](p: Polynomial[R], a: R, n: Nat) {
    polynomial_eval_bound(p, a, n) = R.0 = polynomial_divisible_by_x_minus_a_bound(p, a, n)
}

/// The bounded remainder identity may be stated at any larger bound than a support bound.
theorem polynomial_remainder_theorem_of_support_bounded_le[R: CommRing](
    p: Polynomial[R],
    a: R,
    x: R,
    n: Nat,
    m: Nat
) {
    polynomial_support_bounded_by(p, n) and n <= m implies
    polynomial_eval_bound(p, x, m) =
        polynomial_eval_bound(p, a, n) +
        (x - a) * polynomial_quotient_eval_bound(p, a, x, m)
}

/// A root at one support bound gives divisibility at every larger explicit bound.
theorem polynomial_factor_theorem_forward_of_support_bounded_le[R: CommRing](
    p: Polynomial[R],
    a: R,
    n: Nat,
    m: Nat
) {
    polynomial_support_bounded_by(p, n) and n <= m and polynomial_eval_bound(p, a, n) = R.0
    implies polynomial_divisible_by_x_minus_a_bound(p, a, m)
}

/// Bounded divisibility at a larger explicit bound forces a root at the support bound.
theorem polynomial_factor_theorem_reverse_of_support_bounded_le[R: CommRing](
    p: Polynomial[R],
    a: R,
    n: Nat,
    m: Nat
) {
    polynomial_support_bounded_by(p, n) and n <= m and polynomial_divisible_by_x_minus_a_bound(p, a, m)
    implies polynomial_eval_bound(p, a, n) = R.0
}

/// The bundled bounded remainder identity may be stated at any larger bound than a support bound.
theorem polynomial_remainder_theorem_bundled_of_support_bounded_le[R: CommRing](
    p: Polynomial[R],
    a: R,
    x: R,
    n: Nat,
    m: Nat
) {
    polynomial_support_bounded_by(p, n) and n <= m implies
    polynomial_eval_bound(p, x, m) =
        polynomial_eval_bound(p, a, n) +
        (x - a) * polynomial_eval_bound(polynomial_quotient(p, a, m), x, m)
}

/// A root at one support bound gives bundled divisibility at every larger explicit bound.
theorem polynomial_factor_theorem_forward_bundled_of_support_bounded_le[R: CommRing](
    p: Polynomial[R],
    a: R,
    n: Nat,
    m: Nat
) {
    polynomial_support_bounded_by(p, n) and n <= m and polynomial_eval_bound(p, a, n) = R.0
    implies polynomial_divisible_by_x_minus_a_polynomial_bound(p, a, m)
}

/// Bundled divisibility at a larger explicit bound forces a root at the support bound.
theorem polynomial_factor_theorem_reverse_bundled_of_support_bounded_le[R: CommRing](
    p: Polynomial[R],
    a: R,
    n: Nat,
    m: Nat
) {
    polynomial_support_bounded_by(p, n) and n <= m and polynomial_divisible_by_x_minus_a_polynomial_bound(p, a, m)
    implies polynomial_eval_bound(p, a, n) = R.0
}

/// The coefficient function obtained by applying a semiring homomorphism.
define polynomial_map_coeff[R: Semiring, S: Semiring](
    f: SemiringHom[R, S],
    p: Polynomial[R],
    n: Nat
) -> S {
    f.hom(p.coeff(n))
}

/// The polynomial obtained by applying a semiring homomorphism to every coefficient.
let polynomial_map[R: Semiring, S: Semiring](f: SemiringHom[R, S], p: Polynomial[R]) -> result: Polynomial[S] satisfy {
    Polynomial[S].new(polynomial_map_coeff(f, p)) = Option.some(result)
}

/// A transported polynomial has the transported coefficients.
theorem polynomial_map_coeff_apply[R: Semiring, S: Semiring](
    f: SemiringHom[R, S],
    p: Polynomial[R],
    n: Nat
) {
    polynomial_map(f, p).coeff(n) = f.hom(p.coeff(n))
}

/// A support bound remains valid after transporting coefficients.
theorem polynomial_map_support_bounded_by[R: Semiring, S: Semiring](
    f: SemiringHom[R, S],
    p: Polynomial[R],
    n: Nat
) {
    polynomial_support_bounded_by(p, n) implies
        polynomial_support_bounded_by(polynomial_map(f, p), n)
}

/// Bounded polynomial evaluation commutes with a semiring homomorphism.
theorem polynomial_eval_bound_map[R: Semiring, S: Semiring](
    f: SemiringHom[R, S],
    p: Polynomial[R],
    x: R,
    n: Nat
) {
    polynomial_eval_bound(polynomial_map(f, p), f.hom(x), n) =
        f.hom(polynomial_eval_bound(p, x, n))
}

/// Support-independent polynomial evaluation commutes with a semiring homomorphism.
theorem polynomial_eval_map[R: Semiring, S: Semiring](
    f: SemiringHom[R, S],
    p: Polynomial[R],
    x: R
) {
    polynomial_eval(polynomial_map(f, p), f.hom(x)) = f.hom(polynomial_eval(p, x))
}

/// True if a point is a root of a polynomial.
define is_polynomial_root[R: Semiring](p: Polynomial[R], x: R) -> Bool {
    polynomial_eval(p, x) = R.0
}

/// True if a polynomial value vanishes after applying a semiring homomorphism.
define is_polynomial_root_mod_hom[R: Semiring, S: Semiring](
    f: SemiringHom[R, S],
    p: Polynomial[R],
    x: R
) -> Bool {
    f.hom(polynomial_eval(p, x)) = S.0
}

/// Vanishing modulo a semiring homomorphism is exactly vanishing of the
/// transported polynomial at the transported point.
theorem polynomial_root_mod_hom_iff_mapped_root[R: Semiring, S: Semiring](
    f: SemiringHom[R, S],
    p: Polynomial[R],
    x: R
) {
    is_polynomial_root_mod_hom(f, p, x) =
        is_polynomial_root(polynomial_map(f, p), f.hom(x))
}

/// A quotient formed at a successor bound is supported below the predecessor.
theorem polynomial_quotient_support_bounded_by_pred[F: Field](
    p: Polynomial[F],
    a: F,
    n: Nat
) {
    polynomial_support_bounded_by(polynomial_quotient(p, a, n.suc), n)
}

/// A rooted polynomial is zero if its synthetic quotient is zero at a valid
/// successor support bound.
theorem polynomial_eq_zero_of_root_quotient_eq_zero[F: Field](
    p: Polynomial[F],
    a: F,
    n: Nat
) {
    polynomial_support_bounded_by(p, n.suc) and
        polynomial_eval(p, a) = F.0 and
        polynomial_quotient(p, a, n.suc) = Polynomial[F].zero
    implies p = Polynomial[F].zero
}

/// The synthetic quotient of a nonzero rooted polynomial remains nonzero.
theorem polynomial_quotient_ne_zero_of_root[F: Field](
    p: Polynomial[F],
    a: F,
    n: Nat
) {
    polynomial_support_bounded_by(p, n.suc) and
        p != Polynomial[F].zero and
        polynomial_eval(p, a) = F.0
    implies polynomial_quotient(p, a, n.suc) != Polynomial[F].zero
}

/// A list consists entirely of roots of a polynomial.
define polynomial_roots_on_list[F: Field](p: Polynomial[F], roots: List[F]) -> Bool {
    forall(x: F) {
        roots.contains(x) implies polynomial_eval(p, x) = F.0
    }
}

/// The root property for a list gives the root property for its tail.
theorem polynomial_roots_on_list_tail[F: Field](
    p: Polynomial[F],
    head: F,
    tail: List[F]
) {
    polynomial_roots_on_list(p, List.cons(head, tail)) implies
        polynomial_roots_on_list(p, tail)
}

/// A root distinct from the selected root remains a root of the synthetic quotient.
theorem polynomial_quotient_root_of_distinct_root[F: Field](
    p: Polynomial[F],
    a: F,
    x: F,
    n: Nat
) {
    polynomial_support_bounded_by(p, n) and
        polynomial_eval(p, a) = F.0 and
        polynomial_eval(p, x) = F.0 and
        x != a
    implies polynomial_eval(polynomial_quotient(p, a, n), x) = F.0
}

/// After selecting the head root, every remaining distinct root is a root of
/// the synthetic quotient.
theorem polynomial_roots_on_list_quotient_tail[F: Field](
    p: Polynomial[F],
    head: F,
    tail: List[F],
    n: Nat
) {
    polynomial_support_bounded_by(p, n) and
        List.cons(head, tail).is_unique and
        polynomial_roots_on_list(p, List.cons(head, tail))
    implies polynomial_roots_on_list(polynomial_quotient(p, head, n), tail)
}

/// A nonzero polynomial supported below `n` has fewer than `n` distinct roots
/// in any explicitly listed collection.
theorem polynomial_root_list_length_lt_support_bound[F: Field](
    p: Polynomial[F],
    roots: List[F],
    n: Nat
) {
    polynomial_support_bounded_by(p, n) and
        p != Polynomial[F].zero and
        roots.is_unique and
        polynomial_roots_on_list(p, roots)
    implies roots.length < n
}
