/// Degrees of polynomials: the degree-at-most predicate, the coefficient
/// bounds it imposes, and exact degrees for monomials and sums.

from nat import Nat, lt_suc, lt_imp_lte_suc, lte_imp_not_lt, lte_suc_suc, zero_or_suc,
    add_sub, add_imp_sub, lt_add_left, lte_and_lt, lt_not_ref, lte_add_right, lte_trans,
    lt_or_lte, trichotomy, lte_antisymm
from order import lt_imp_lte, lte_max_left, lte_max_right
from semiring import Semiring
from comm_ring import CommRing
from algebra.ring.ring import Ring
from algebra.add_group import inverse_left, inverse_inverse
from algebra.field.field import Field, field_mul_nonzero
from polynomial import Polynomial, polynomial_constant, polynomial_monomial,
    polynomial_monomial_coeff_self, polynomial_monomial_coeff_of_ne,
    polynomial_constant_coeff_zero, polynomial_constant_coeff_of_ne_zero,
    polynomial_zero_coeff, polynomial_ext_pointwise, polynomial_add_coeff,
    polynomial_neg_coeff, polynomial_support_bounded_by, polynomial_support_bounded_by_monotone,
    polynomial_support_bounded_by_apply, polynomial_add_support_bounded_by,
    polynomial_zero_support_bounded_by_zero, polynomial_constant_support_bounded_by_one,
    polynomial_mul, polynomial_mul_support_bounded_by_add, polynomial_mul_coeff,
    polynomial_mul_coeff_apply, polynomial_mul_term_coeff, coeff_zero_from, coeff_zero_at,
    coeff_zero_from_at, coeff_zero_from_apply
from polynomial.polynomial_eval_deep import polynomial_monomial_support_bounded_by_suc
from polynomial.polynomial_monomial import polynomial_monomial_mul
from polynomial_mul_comm import polynomial_mul_comm, mul_coeff_eq_range_sum
from data.nat.nat_range_sum import range_sum, range_sum_congr
from data.nat.nat_residue_range_sum import point_fn, point_fn_at, point_fn_off, range_sum_point_fn,
    range_sum_zero_of_vanishes
from data.nat.nat_range_sum_bridge import range_sum_eq_partial
from list import partial

numerals Nat

/// The degree of a polynomial is at most `d` when its support lies below
/// `d + 1`.
define polynomial_degree_le[R: Semiring](p: Polynomial[R], d: Nat) -> Bool {
    polynomial_support_bounded_by(p, d.suc)
}

/// The attribute monomial is the same polynomial as the named monomial.
lemma degree_monomial_alias[R: Semiring](n: Nat, r: R) {
    Polynomial[R].monomial(n, r) = polynomial_monomial(n, r)
} by {
    Polynomial[R].monomial(n, r) = polynomial_monomial(n, r)
}

/// A monomial has its coefficient at its exponent.
lemma degree_monomial_coeff_self[R: Semiring](n: Nat, r: R) {
    polynomial_monomial(n, r).coeff(n) = r
} by {
    degree_monomial_alias(n, r)
    Polynomial[R].monomial(n, r) = polynomial_monomial(n, r)
    polynomial_monomial_coeff_self(n, r)
    Polynomial[R].monomial(n, r).coeff(n) = r
    polynomial_monomial(n, r).coeff(n) = r
}

/// A monomial vanishes away from its exponent.
lemma degree_monomial_coeff_of_ne[R: Semiring](n: Nat, r: R, k: Nat) {
    k != n implies polynomial_monomial(n, r).coeff(k) = R.0
} by {
    if k != n {
        degree_monomial_alias(n, r)
        Polynomial[R].monomial(n, r) = polynomial_monomial(n, r)
        polynomial_monomial_coeff_of_ne[R](n, r, k)
        Polynomial[R].monomial(n, r).coeff(k) = R.0
        polynomial_monomial(n, r).coeff(k) = R.0
    }
}

/// A nonzero exponent is positive.
lemma degree_nat_zero_lt_of_suc(kp: Nat) {
    Nat.0 < kp.suc
} by {
    Nat.0 < kp.suc
}

/// A successor of a successor is at least two.
lemma degree_nat_one_lt_of_suc_suc(kq: Nat) {
    Nat.1 < kq.suc.suc
} by {
    Nat.1 < kq.suc.suc
}

/// The zero polynomial has degree at most any natural number.
theorem polynomial_degree_le_zero[R: Semiring](d: Nat) {
    polynomial_degree_le(Polynomial[R].zero, d)
} by {
    polynomial_degree_le(Polynomial[R].zero, d) = polynomial_support_bounded_by(Polynomial[R].zero, d.suc)
    polynomial_zero_support_bounded_by_zero[R]
    polynomial_support_bounded_by(Polynomial[R].zero, Nat.0)
    Nat.0 <= d.suc
    polynomial_support_bounded_by_monotone(Polynomial[R].zero, Nat.0, d.suc)
    polynomial_support_bounded_by(Polynomial[R].zero, d.suc)
    polynomial_degree_le(Polynomial[R].zero, d)
}

/// A constant polynomial has degree at most zero.
theorem polynomial_degree_constant_le_zero[R: Semiring](r: R) {
    polynomial_degree_le(polynomial_constant(r), Nat.0)
} by {
    polynomial_degree_le(polynomial_constant(r), Nat.0) =
        polynomial_support_bounded_by(polynomial_constant(r), Nat.0.suc)
    polynomial_constant_support_bounded_by_one(r)
    polynomial_support_bounded_by(polynomial_constant(r), Nat.0.suc)
    polynomial_degree_le(polynomial_constant(r), Nat.0)
}

/// A monomial of exponent `n` has degree at most `n`.
theorem polynomial_degree_monomial_le[R: Semiring](n: Nat, r: R) {
    polynomial_degree_le(polynomial_monomial(n, r), n)
} by {
    polynomial_degree_le(polynomial_monomial(n, r), n) =
        polynomial_support_bounded_by(polynomial_monomial(n, r), n.suc)
    polynomial_monomial_support_bounded_by_suc(n, r)
    polynomial_support_bounded_by(polynomial_monomial(n, r), n.suc)
    polynomial_degree_le(polynomial_monomial(n, r), n)
}

/// A degree bound can be raised.
theorem polynomial_degree_le_monotone[R: Semiring](p: Polynomial[R], d: Nat, e: Nat) {
    polynomial_degree_le(p, d) and d <= e implies polynomial_degree_le(p, e)
} by {
    if polynomial_degree_le(p, d) and d <= e {
        polynomial_degree_le(p, d) = polynomial_support_bounded_by(p, d.suc)
        polynomial_support_bounded_by(p, d.suc)
        lte_suc_suc(d, e)
        d.suc <= e.suc
        polynomial_support_bounded_by_monotone(p, d.suc, e.suc)
        polynomial_support_bounded_by(p, e.suc)
        polynomial_degree_le(p, e) = polynomial_support_bounded_by(p, e.suc)
        polynomial_degree_le(p, e)
    }
}

/// A coefficient beyond the degree bound vanishes.
theorem polynomial_coeff_zero_of_degree_lt[R: Semiring](p: Polynomial[R], d: Nat, n: Nat) {
    polynomial_degree_le(p, d) and d < n implies p.coeff(n) = R.0
} by {
    if polynomial_degree_le(p, d) and d < n {
        polynomial_degree_le(p, d) = polynomial_support_bounded_by(p, d.suc)
        polynomial_support_bounded_by(p, d.suc)
        lt_imp_lte_suc(d, n)
        d.suc <= n
        lte_imp_not_lt(d.suc, n)
        not n < d.suc
        polynomial_support_bounded_by_apply(p, d.suc, n)
        p.coeff(n) = R.0
    }
}

/// A nonzero coefficient excludes the degree bound below its exponent.
theorem polynomial_coeff_nonzero_not_degree_lt[R: Semiring](
    p: Polynomial[R], n: Nat, d: Nat
) {
    p.coeff(n) != R.0 implies not (polynomial_degree_le(p, d) and d < n)
} by {
    if p.coeff(n) != R.0 {
        if polynomial_degree_le(p, d) and d < n {
            polynomial_coeff_zero_of_degree_lt(p, d, n)
            p.coeff(n) = R.0
            false
        }
        not (polynomial_degree_le(p, d) and d < n)
    }
}

/// A polynomial of degree at most zero is a constant polynomial.
theorem polynomial_degree_le_zero_iff_constant[R: Semiring](p: Polynomial[R]) {
    polynomial_degree_le(p, Nat.0) iff p = polynomial_constant(p.coeff(Nat.0))
} by {
    if polynomial_degree_le(p, Nat.0) {
        forall(k: Nat) {
            if k = Nat.0 {
                polynomial_constant_coeff_zero(p.coeff(Nat.0))
                polynomial_constant(p.coeff(Nat.0)).coeff(k) = p.coeff(Nat.0)
                p.coeff(k) = p.coeff(Nat.0)
                p.coeff(k) = polynomial_constant(p.coeff(Nat.0)).coeff(k)
            } else {
                k != Nat.0
                zero_or_suc(k)
                let kp: Nat satisfy {
                    k = kp.suc
                }
                k = kp.suc
                degree_nat_zero_lt_of_suc(kp)
                Nat.0 < k
                polynomial_coeff_zero_of_degree_lt(p, Nat.0, k)
                p.coeff(k) = R.0
                polynomial_constant_coeff_of_ne_zero(p.coeff(Nat.0), k)
                polynomial_constant(p.coeff(Nat.0)).coeff(k) = R.0
                p.coeff(k) = polynomial_constant(p.coeff(Nat.0)).coeff(k)
            }
        }
        polynomial_ext_pointwise(p, polynomial_constant(p.coeff(Nat.0)))
        p = polynomial_constant(p.coeff(Nat.0))
    }
    if p = polynomial_constant(p.coeff(Nat.0)) {
        polynomial_degree_constant_le_zero(p.coeff(Nat.0))
        polynomial_degree_le(polynomial_constant(p.coeff(Nat.0)), Nat.0)
        p = polynomial_constant(p.coeff(Nat.0))
        polynomial_degree_le(p, Nat.0)
    }
}

/// A polynomial of degree at most one is the sum of its constant and linear
/// parts.
theorem polynomial_degree_le_one_iff_linear[R: Semiring](p: Polynomial[R]) {
    polynomial_degree_le(p, Nat.1) iff
        p = polynomial_constant(p.coeff(Nat.0)) + polynomial_monomial(Nat.1, p.coeff(Nat.1))
} by {
    if polynomial_degree_le(p, Nat.1) {
        forall(k: Nat) {
            if k = Nat.0 {
                polynomial_add_coeff(polynomial_constant(p.coeff(Nat.0)),
                    polynomial_monomial(Nat.1, p.coeff(Nat.1)), k)
                (polynomial_constant(p.coeff(Nat.0)) + polynomial_monomial(Nat.1, p.coeff(Nat.1))).coeff(k) =
                    polynomial_constant(p.coeff(Nat.0)).coeff(k) +
                    polynomial_monomial(Nat.1, p.coeff(Nat.1)).coeff(k)
                polynomial_constant_coeff_zero(p.coeff(Nat.0))
                polynomial_constant(p.coeff(Nat.0)).coeff(k) = p.coeff(Nat.0)
                Nat.0 != Nat.1
                degree_monomial_coeff_of_ne(Nat.1, p.coeff(Nat.1), Nat.0)
                polynomial_monomial(Nat.1, p.coeff(Nat.1)).coeff(k) = R.0
                (polynomial_constant(p.coeff(Nat.0)) + polynomial_monomial(Nat.1, p.coeff(Nat.1))).coeff(k) = p.coeff(Nat.0) + R.0
                p.coeff(Nat.0) + R.0 = p.coeff(Nat.0)
                p.coeff(k) = p.coeff(Nat.0)
                p.coeff(k) = (polynomial_constant(p.coeff(Nat.0)) +
                    polynomial_monomial(Nat.1, p.coeff(Nat.1))).coeff(k)
            } else {
                k != Nat.0
                zero_or_suc(k)
                let kp: Nat satisfy {
                    k = kp.suc
                }
                k = kp.suc
                if kp = Nat.0 {
                    k = Nat.1
                    polynomial_add_coeff(polynomial_constant(p.coeff(Nat.0)),
                        polynomial_monomial(Nat.1, p.coeff(Nat.1)), k)
                    (polynomial_constant(p.coeff(Nat.0)) +
                        polynomial_monomial(Nat.1, p.coeff(Nat.1))).coeff(k) = polynomial_constant(p.coeff(Nat.0)).coeff(k) +
                        polynomial_monomial(Nat.1, p.coeff(Nat.1)).coeff(k)
                    Nat.1 != Nat.0
                    polynomial_constant_coeff_of_ne_zero(p.coeff(Nat.0), Nat.1)
                    polynomial_constant(p.coeff(Nat.0)).coeff(k) = R.0
                    degree_monomial_coeff_self(Nat.1, p.coeff(Nat.1))
                    polynomial_monomial(Nat.1, p.coeff(Nat.1)).coeff(k) = p.coeff(Nat.1)
                    (polynomial_constant(p.coeff(Nat.0)) +
                        polynomial_monomial(Nat.1, p.coeff(Nat.1))).coeff(k) =
                        R.0 + p.coeff(Nat.1)
                    R.0 + p.coeff(Nat.1) = p.coeff(Nat.1)
                    p.coeff(k) = p.coeff(Nat.1)
                    p.coeff(k) = (polynomial_constant(p.coeff(Nat.0)) +
                        polynomial_monomial(Nat.1, p.coeff(Nat.1))).coeff(k)
                } else {
                    kp != Nat.0
                    zero_or_suc(kp)
                    let kq: Nat satisfy {
                        kp = kq.suc
                    }
                    kp = kq.suc
                    k = kq.suc.suc
                    degree_nat_one_lt_of_suc_suc(kq)
                    Nat.1 < k
                    polynomial_coeff_zero_of_degree_lt(p, Nat.1, k)
                    p.coeff(k) = R.0
                    polynomial_add_coeff(polynomial_constant(p.coeff(Nat.0)),
                        polynomial_monomial(Nat.1, p.coeff(Nat.1)), k)
                    (polynomial_constant(p.coeff(Nat.0)) +
                        polynomial_monomial(Nat.1, p.coeff(Nat.1))).coeff(k) =
                        polynomial_constant(p.coeff(Nat.0)).coeff(k) +
                        polynomial_monomial(Nat.1, p.coeff(Nat.1)).coeff(k)
                    polynomial_constant_coeff_of_ne_zero(p.coeff(Nat.0), k)
                    polynomial_constant(p.coeff(Nat.0)).coeff(k) = R.0
                    k != Nat.1
                    degree_monomial_coeff_of_ne(Nat.1, p.coeff(Nat.1), k)
                    polynomial_monomial(Nat.1, p.coeff(Nat.1)).coeff(k) = R.0
                    (polynomial_constant(p.coeff(Nat.0)) +
                        polynomial_monomial(Nat.1, p.coeff(Nat.1))).coeff(k) = R.0 + R.0
                    R.0 + R.0 = R.0
                    p.coeff(k) = (polynomial_constant(p.coeff(Nat.0)) +
                        polynomial_monomial(Nat.1, p.coeff(Nat.1))).coeff(k)
                }
            }
        }
        polynomial_ext_pointwise(p, polynomial_constant(p.coeff(Nat.0)) +
            polynomial_monomial(Nat.1, p.coeff(Nat.1)))
        p = polynomial_constant(p.coeff(Nat.0)) + polynomial_monomial(Nat.1, p.coeff(Nat.1))
    }
    if p = polynomial_constant(p.coeff(Nat.0)) + polynomial_monomial(Nat.1, p.coeff(Nat.1)) {
        polynomial_constant_support_bounded_by_one(p.coeff(Nat.0))
        polynomial_support_bounded_by(polynomial_constant(p.coeff(Nat.0)), Nat.1)
        lt_suc(Nat.1)
        Nat.1 < Nat.1.suc
        lt_imp_lte[Nat](Nat.1, Nat.1.suc)
        Nat.1 <= Nat.2
        polynomial_support_bounded_by_monotone(polynomial_constant(p.coeff(Nat.0)), Nat.1, Nat.2)
        polynomial_support_bounded_by(polynomial_constant(p.coeff(Nat.0)), Nat.2)
        polynomial_monomial_support_bounded_by_suc(Nat.1, p.coeff(Nat.1))
        polynomial_support_bounded_by(polynomial_monomial(Nat.1, p.coeff(Nat.1)), Nat.2)
        polynomial_add_support_bounded_by(polynomial_constant(p.coeff(Nat.0)),
            polynomial_monomial(Nat.1, p.coeff(Nat.1)), Nat.2)
        polynomial_support_bounded_by(polynomial_constant(p.coeff(Nat.0)) +
            polynomial_monomial(Nat.1, p.coeff(Nat.1)), Nat.2)
        polynomial_degree_le(polynomial_constant(p.coeff(Nat.0)) +
            polynomial_monomial(Nat.1, p.coeff(Nat.1)), Nat.1) =
            polynomial_support_bounded_by(polynomial_constant(p.coeff(Nat.0)) +
                polynomial_monomial(Nat.1, p.coeff(Nat.1)), Nat.1.suc)
        polynomial_degree_le(polynomial_constant(p.coeff(Nat.0)) +
            polynomial_monomial(Nat.1, p.coeff(Nat.1)), Nat.1)
        p = polynomial_constant(p.coeff(Nat.0)) + polynomial_monomial(Nat.1, p.coeff(Nat.1))
        polynomial_degree_le(p, Nat.1)
    }
}

/// The classical bound for the degree of a sum: the degree of a sum is at
/// most the maximum of the degrees.
theorem polynomial_degree_sum_le_max[R: Semiring](
    p: Polynomial[R], q: Polynomial[R], m: Nat, n: Nat
) {
    polynomial_degree_le(p, m) and polynomial_degree_le(q, n) implies
    polynomial_degree_le(p + q, m.max(n))
} by {
    if polynomial_degree_le(p, m) and polynomial_degree_le(q, n) {
        polynomial_degree_le(p, m) = polynomial_support_bounded_by(p, m.suc)
        polynomial_support_bounded_by(p, m.suc)
        lte_max_left(m, n)
        m <= m.max(n)
        lte_suc_suc(m, m.max(n))
        m.suc <= m.max(n).suc
        polynomial_support_bounded_by_monotone(p, m.suc, m.max(n).suc)
        polynomial_support_bounded_by(p, m.max(n).suc)
        polynomial_degree_le(q, n) = polynomial_support_bounded_by(q, n.suc)
        polynomial_support_bounded_by(q, n.suc)
        lte_max_right(m, n)
        n <= m.max(n)
        lte_suc_suc(n, m.max(n))
        n.suc <= m.max(n).suc
        polynomial_support_bounded_by_monotone(q, n.suc, m.max(n).suc)
        polynomial_support_bounded_by(q, m.max(n).suc)
        polynomial_add_support_bounded_by(p, q, m.max(n).suc)
        polynomial_support_bounded_by(p + q, m.max(n).suc)
        polynomial_degree_le(p + q, m.max(n)) =
            polynomial_support_bounded_by(p + q, m.max(n).suc)
        polynomial_degree_le(p + q, m.max(n))
    }
}

/// When the degrees differ, the leading coefficient of the sum is the leading
/// coefficient of the higher-degree summand.
theorem polynomial_degree_sum_leading_coeff[R: Semiring](
    p: Polynomial[R], q: Polynomial[R], m: Nat, n: Nat
) {
    polynomial_degree_le(p, m) and polynomial_degree_le(q, n) and m < n implies
    (p + q).coeff(n) = q.coeff(n)
} by {
    if polynomial_degree_le(p, m) and polynomial_degree_le(q, n) and m < n {
        polynomial_coeff_zero_of_degree_lt(p, m, n)
        p.coeff(n) = R.0
        polynomial_add_coeff(p, q, n)
        (p + q).coeff(n) = p.coeff(n) + q.coeff(n)
        (p + q).coeff(n) = R.0 + q.coeff(n)
        R.0 + q.coeff(n) = q.coeff(n)
        (p + q).coeff(n) = q.coeff(n)
    }
}

/// A monomial with nonzero coefficient has exact degree `n`: no degree bound
/// below `n` applies.
theorem polynomial_degree_monomial_exact[R: Semiring](n: Nat, r: R, k: Nat) {
    r != R.0 and k < n implies not polynomial_degree_le(polynomial_monomial(n, r), k)
} by {
    if r != R.0 and k < n {
        if polynomial_degree_le(polynomial_monomial(n, r), k) {
            polynomial_degree_le(polynomial_monomial(n, r), k) =
                polynomial_support_bounded_by(polynomial_monomial(n, r), k.suc)
            polynomial_support_bounded_by(polynomial_monomial(n, r), k.suc)
            lt_imp_lte_suc(k, n)
            k.suc <= n
            lte_imp_not_lt(k.suc, n)
            not n < k.suc
            polynomial_support_bounded_by_apply(polynomial_monomial(n, r), k.suc, n)
            polynomial_monomial(n, r).coeff(n) = R.0
            degree_monomial_coeff_self(n, r)
            polynomial_monomial(n, r).coeff(n) = r
            r = R.0
            false
        }
        not polynomial_degree_le(polynomial_monomial(n, r), k)
    }
}

/// Over a field, the product of two monomials with nonzero coefficients has
/// exact degree `m + n`.
theorem polynomial_degree_monomial_mul_exact[F: Field](m: Nat, a: F, n: Nat, b: F, k: Nat) {
    a != F.0 and b != F.0 and k < m + n implies
    not polynomial_degree_le(polynomial_mul(polynomial_monomial(m, a),
        polynomial_monomial(n, b)), k)
} by {
    if a != F.0 and b != F.0 and k < m + n {
        polynomial_monomial_mul(m, a, n, b)
        polynomial_mul(polynomial_monomial(m, a), polynomial_monomial(n, b)) =
            polynomial_monomial(m + n, a * b)
        field_mul_nonzero(a, b)
        a * b != F.0
        polynomial_degree_monomial_exact(m + n, a * b, k)
        not polynomial_degree_le(polynomial_monomial(m + n, a * b), k)
        if polynomial_degree_le(polynomial_mul(polynomial_monomial(m, a),
            polynomial_monomial(n, b)), k) {
            polynomial_degree_le(polynomial_monomial(m + n, a * b), k)
            false
        }
        not polynomial_degree_le(polynomial_mul(polynomial_monomial(m, a),
            polynomial_monomial(n, b)), k)
    }
}

/// Over a field, a sum whose higher-degree summand has nonzero leading
/// coefficient has exact degree `n`.
theorem polynomial_degree_sum_exact[F: Field](
    p: Polynomial[F], q: Polynomial[F], m: Nat, n: Nat, k: Nat
) {
    polynomial_degree_le(p, m) and polynomial_degree_le(q, n) and m < n and k < n and
    q.coeff(n) != F.0 implies not polynomial_degree_le(p + q, k)
} by {
    if polynomial_degree_le(p, m) and polynomial_degree_le(q, n) and m < n and k < n and
        q.coeff(n) != F.0 {
        polynomial_degree_sum_leading_coeff(p, q, m, n)
        (p + q).coeff(n) = q.coeff(n)
        if polynomial_degree_le(p + q, k) {
            polynomial_coeff_zero_of_degree_lt(p + q, k, n)
            (p + q).coeff(n) = F.0
            q.coeff(n) = F.0
            false
        }
        not polynomial_degree_le(p + q, k)
    }
}

/// The degree of a monomial product is at most the sum of the degrees.
theorem polynomial_degree_monomial_mul_le[R: CommRing](m: Nat, a: R, n: Nat, b: R) {
    polynomial_degree_le(polynomial_mul(polynomial_monomial(m, a), polynomial_monomial(n, b)),
        m + n)
} by {
    polynomial_monomial_mul(m, a, n, b)
    polynomial_mul(polynomial_monomial(m, a), polynomial_monomial(n, b)) =
        polynomial_monomial(m + n, a * b)
    polynomial_degree_monomial_le(m + n, a * b)
    polynomial_degree_le(polynomial_monomial(m + n, a * b), m + n)
    polynomial_degree_le(polynomial_mul(polynomial_monomial(m, a), polynomial_monomial(n, b)),
        m + n)
}

/// The classical bound for the degree of a product: the degree of a product
/// is at most the sum of the degrees.
theorem polynomial_degree_mul_le_add[R: Semiring](
    p: Polynomial[R], q: Polynomial[R], m: Nat, n: Nat
) {
    polynomial_degree_le(p, m) and polynomial_degree_le(q, n) implies
    polynomial_degree_le(polynomial_mul(p, q), m + n + Nat.1)
} by {
    if polynomial_degree_le(p, m) and polynomial_degree_le(q, n) {
        polynomial_degree_le(p, m) = polynomial_support_bounded_by(p, m.suc)
        polynomial_support_bounded_by(p, m.suc)
        polynomial_degree_le(q, n) = polynomial_support_bounded_by(q, n.suc)
        polynomial_support_bounded_by(q, n.suc)
        polynomial_mul_support_bounded_by_add(p, q, m.suc, n.suc)
        polynomial_support_bounded_by(polynomial_mul(p, q), m.suc + n.suc)
        (m + n + Nat.1).suc = m.suc + n.suc
        polynomial_support_bounded_by(polynomial_mul(p, q), (m + n + Nat.1).suc)
        polynomial_degree_le(polynomial_mul(p, q), m + n + Nat.1) =
            polynomial_support_bounded_by(polynomial_mul(p, q), (m + n + Nat.1).suc)
        polynomial_degree_le(polynomial_mul(p, q), m + n + Nat.1)
    }
}

/// The degree bound of a product is unchanged by swapping the factors.
theorem polynomial_degree_mul_comm_le[R: CommRing](
    p: Polynomial[R], q: Polynomial[R], d: Nat
) {
    (polynomial_degree_le(polynomial_mul(p, q), d)) =
        (polynomial_degree_le(polynomial_mul(q, p), d))
} by {
    polynomial_mul_comm(p, q)
    polynomial_mul(p, q) = polynomial_mul(q, p)
    (polynomial_degree_le(polynomial_mul(p, q), d)) =
        (polynomial_degree_le(polynomial_mul(q, p), d))
}

/// Over a field, the power `X^n` has exact degree `n`.
theorem polynomial_degree_x_pow[F: Field](n: Nat, k: Nat) {
    k < n implies not polynomial_degree_le(polynomial_monomial(n, F.1), k)
} by {
    if k < n {
        F.1 != F.0
        polynomial_degree_monomial_exact(n, F.1, k)
        not polynomial_degree_le(polynomial_monomial(n, F.1), k)
    }
}

/// `m <= m + n` for natural numbers.
lemma degree_nat_lte_add(m: Nat, n: Nat) {
    m <= m + n
} by {
    m + n = m + n
}

/// Subtracting `m` from `m + n` leaves `n`.
lemma degree_nat_sub_add(m: Nat, n: Nat) {
    (m + n) - m = n
} by {
    n + m = m + n
    add_imp_sub(n, m, m + n)
    (m + n) - m = n
}

/// Below `m`, the complement `m + n - i` stays at least `n + 1`.
lemma degree_nat_sub_gt(m: Nat, n: Nat, i: Nat) {
    i < m implies not (m + n - i) < n.suc
} by {
    if i < m {
        if (m + n - i) < n.suc {
            (m + n - i) <= n
            i <= m
            degree_nat_lte_add(m, n)
            m <= m + n
            lte_trans(i, m, m + n)
            i <= m + n
            add_sub(m + n, i)
            (m + n - i) + i = m + n
            lte_add_right(i, m + n - i, n)
            (m + n - i) + i <= n + i
            m + n <= n + i
            lt_add_left(n, i, m)
            n + i < n + m
            n + m = m + n
            n + i < m + n
            lte_and_lt(m + n, n + i, m + n)
            m + n < m + n
            lt_not_ref(m + n)
            not (m + n < m + n)
            false
        }
        not (m + n - i) < n.suc
    }
}

/// The leading coefficient of a product is the product of the leading
/// coefficients: the coefficient of `X^(m + n)` in `p * q`.
theorem polynomial_mul_leading_coeff[R: CommRing](
    p: Polynomial[R], q: Polynomial[R], m: Nat, n: Nat
) {
    polynomial_degree_le(p, m) and polynomial_degree_le(q, n) implies
    polynomial_mul(p, q).coeff(m + n) = p.coeff(m) * q.coeff(n)
} by {
    if polynomial_degree_le(p, m) and polynomial_degree_le(q, n) {
        polynomial_mul_coeff_apply(p, q, m + n)
        polynomial_mul(p, q).coeff(m + n) = polynomial_mul_coeff(p, q, m + n)
        mul_coeff_eq_range_sum(p, q, m + n)
        polynomial_mul_coeff(p, q, m + n) =
            range_sum(polynomial_mul_term_coeff(p, q, m + n), (m + n).suc)
        forall(i: Nat) {
            if i < (m + n).suc {
                if i = m {
                    i = m
                    polynomial_mul_term_coeff(p, q, m + n, i) =
                        p.coeff(i) * q.coeff((m + n) - i)
                    (m + n) - i = (m + n) - m
                    degree_nat_sub_add(m, n)
                    (m + n) - m = n
                    (m + n) - i = n
                    polynomial_mul_term_coeff(p, q, m + n, i) = p.coeff(m) * q.coeff(n)
                    point_fn_at(m, p.coeff(m) * q.coeff(n))
                    point_fn(m, p.coeff(m) * q.coeff(n))(i) = p.coeff(m) * q.coeff(n)
                    polynomial_mul_term_coeff(p, q, m + n)(i) =
                        point_fn(m, p.coeff(m) * q.coeff(n))(i)
                } else {
                    i != m
                    if i < m {
                        degree_nat_sub_gt(m, n, i)
                        not (m + n - i) < n.suc
                        polynomial_degree_le(q, n) = polynomial_support_bounded_by(q, n.suc)
                        polynomial_support_bounded_by(q, n.suc)
                        polynomial_support_bounded_by_apply(q, n.suc, (m + n) - i)
                        q.coeff((m + n) - i) = R.0
                        polynomial_mul_term_coeff(p, q, m + n, i) =
                            p.coeff(i) * q.coeff((m + n) - i)
                        polynomial_mul_term_coeff(p, q, m + n, i) = p.coeff(i) * R.0
                        p.coeff(i) * R.0 = R.0
                        point_fn_off(m, p.coeff(m) * q.coeff(n), i)
                        point_fn(m, p.coeff(m) * q.coeff(n))(i) = R.0
                        polynomial_mul_term_coeff(p, q, m + n)(i) =
                            point_fn(m, p.coeff(m) * q.coeff(n))(i)
                    } else {
                        not i < m
                        trichotomy(i, m)
                        if i < m {
                            false
                        }
                        if i = m {
                            false
                        }
                        m < i
                        lt_imp_lte_suc(m, i)
                        m.suc <= i
                        lte_imp_not_lt(m.suc, i)
                        not i < m.suc
                        polynomial_degree_le(p, m) = polynomial_support_bounded_by(p, m.suc)
                        polynomial_support_bounded_by(p, m.suc)
                        polynomial_support_bounded_by_apply(p, m.suc, i)
                        p.coeff(i) = R.0
                        polynomial_mul_term_coeff(p, q, m + n, i) =
                            p.coeff(i) * q.coeff((m + n) - i)
                        polynomial_mul_term_coeff(p, q, m + n, i) = R.0 * q.coeff((m + n) - i)
                        R.0 * q.coeff((m + n) - i) = R.0
                        point_fn_off(m, p.coeff(m) * q.coeff(n), i)
                        point_fn(m, p.coeff(m) * q.coeff(n))(i) = R.0
                        polynomial_mul_term_coeff(p, q, m + n)(i) =
                            point_fn(m, p.coeff(m) * q.coeff(n))(i)
                    }
                }
            }
        }
        range_sum_congr(polynomial_mul_term_coeff(p, q, m + n),
            point_fn(m, p.coeff(m) * q.coeff(n)), (m + n).suc)
        range_sum(polynomial_mul_term_coeff(p, q, m + n), (m + n).suc) =
            range_sum(point_fn(m, p.coeff(m) * q.coeff(n)), (m + n).suc)
        degree_nat_lte_add(m, n)
        m <= m + n
        lt_suc(m + n)
        m + n < (m + n).suc
        lte_and_lt(m, m + n, (m + n).suc)
        m < (m + n).suc
        range_sum_point_fn(m, p.coeff(m) * q.coeff(n), (m + n).suc)
        range_sum(point_fn(m, p.coeff(m) * q.coeff(n)), (m + n).suc) =
            p.coeff(m) * q.coeff(n)
        polynomial_mul_coeff(p, q, m + n) = p.coeff(m) * q.coeff(n)
        polynomial_mul(p, q).coeff(m + n) = p.coeff(m) * q.coeff(n)
    }
}

/// Over a field, a product with nonzero leading coefficients has degree at
/// least the sum of the degrees.
theorem polynomial_degree_mul_lower_bound[F: Field](
    p: Polynomial[F], q: Polynomial[F], m: Nat, n: Nat, k: Nat
) {
    polynomial_degree_le(p, m) and polynomial_degree_le(q, n) and
    p.coeff(m) != F.0 and q.coeff(n) != F.0 and k < m + n implies
    not polynomial_degree_le(polynomial_mul(p, q), k)
} by {
    if polynomial_degree_le(p, m) and polynomial_degree_le(q, n) and
        p.coeff(m) != F.0 and q.coeff(n) != F.0 and k < m + n {
        polynomial_mul_leading_coeff(p, q, m, n)
        polynomial_mul(p, q).coeff(m + n) = p.coeff(m) * q.coeff(n)
        field_mul_nonzero(p.coeff(m), q.coeff(n))
        p.coeff(m) * q.coeff(n) != F.0
        polynomial_mul(p, q).coeff(m + n) != F.0
        polynomial_coeff_nonzero_not_degree_lt(polynomial_mul(p, q), m + n, k)
        not (polynomial_degree_le(polynomial_mul(p, q), k) and k < m + n)
        if polynomial_degree_le(polynomial_mul(p, q), k) {
            false
        }
        not polynomial_degree_le(polynomial_mul(p, q), k)
    }
}

/// Over a field, the degree of a product with nonzero leading coefficients
/// is exactly the sum of the degrees.
theorem polynomial_degree_mul_exact[F: Field](
    p: Polynomial[F], q: Polynomial[F], m: Nat, n: Nat
) {
    polynomial_degree_le(p, m) and polynomial_degree_le(q, n) and
    p.coeff(m) != F.0 and q.coeff(n) != F.0
    implies (polynomial_degree_le(polynomial_mul(p, q), m + n + Nat.1) and
        forall(k: Nat) {
            k < m + n implies not polynomial_degree_le(polynomial_mul(p, q), k)
        })
} by {
    if polynomial_degree_le(p, m) and polynomial_degree_le(q, n) and
        p.coeff(m) != F.0 and q.coeff(n) != F.0 {
        polynomial_degree_mul_le_add(p, q, m, n)
        polynomial_degree_le(polynomial_mul(p, q), m + n + Nat.1)
        forall(k: Nat) {
            if k < m + n {
                polynomial_degree_mul_lower_bound(p, q, m, n, k)
                not polynomial_degree_le(polynomial_mul(p, q), k)
            }
        }
        polynomial_degree_le(polynomial_mul(p, q), m + n + Nat.1) and
            forall(k: Nat) {
                k < m + n implies not polynomial_degree_le(polynomial_mul(p, q), k)
            }
    }
}

/// Dropping a zero top coefficient lowers the degree bound: if a polynomial
/// has degree at most d + 1 and its coefficient at d + 1 vanishes, then it has
/// degree at most d.
theorem polynomial_degree_drop_top[R: Semiring](p: Polynomial[R], d: Nat) {
    polynomial_degree_le(p, d.suc) and p.coeff(d.suc) = R.0 implies polynomial_degree_le(p, d)
} by {
    if polynomial_degree_le(p, d.suc) and p.coeff(d.suc) = R.0 {
        polynomial_degree_le(p, d.suc) = polynomial_support_bounded_by(p, d.suc.suc)
        polynomial_support_bounded_by(p, d.suc.suc)
        polynomial_support_bounded_by(p, d.suc.suc) = coeff_zero_from(p.coeff, d.suc.suc)
        coeff_zero_from(p.coeff, d.suc.suc)
        forall(k: Nat) {
            if not k < d.suc {
                if k = d.suc {
                    p.coeff(d.suc) = R.0
                    p.coeff(k) = R.0
                } else {
                    k != d.suc
                    not k < d.suc
                    trichotomy(k, d.suc)
                    if k < d.suc {
                        false
                    }
                    if k = d.suc {
                        false
                    }
                    d.suc < k
                    lt_imp_lte_suc(d.suc, k)
                    d.suc.suc <= k
                    lte_imp_not_lt(d.suc.suc, k)
                    not k < d.suc.suc
                    coeff_zero_from_apply(p.coeff, d.suc.suc, k)
                    p.coeff(k) = R.0
                }
                coeff_zero_at(p.coeff, k)
                coeff_zero_from_at(p.coeff, d.suc, k) = (not k < d.suc implies coeff_zero_at(p.coeff, k))
                coeff_zero_from_at(p.coeff, d.suc, k)
            }
        }
        coeff_zero_from(p.coeff, d.suc) = forall(k: Nat) { coeff_zero_from_at(p.coeff, d.suc, k) }
        coeff_zero_from(p.coeff, d.suc)
        polynomial_support_bounded_by(p, d.suc) = coeff_zero_from(p.coeff, d.suc)
        polynomial_support_bounded_by(p, d.suc)
        polynomial_degree_le(p, d) = polynomial_support_bounded_by(p, d.suc)
        polynomial_degree_le(p, d)
    }
}

/// Dropping a zero top coefficient preserves a support bound: if a
/// coefficient function vanishes from the bound n + 1 onward and the
/// coefficient at n vanishes, then it vanishes from n onward.
theorem polynomial_support_bounded_by_drop_top[R: Semiring](p: Polynomial[R], n: Nat) {
    polynomial_support_bounded_by(p, n.suc) and p.coeff(n) = R.0 implies
    polynomial_support_bounded_by(p, n)
} by {
    if polynomial_support_bounded_by(p, n.suc) and p.coeff(n) = R.0 {
        polynomial_support_bounded_by(p, n.suc) = coeff_zero_from(p.coeff, n.suc)
        coeff_zero_from(p.coeff, n.suc)
        forall(k: Nat) {
            if not k < n {
                if k = n {
                    p.coeff(n) = R.0
                    p.coeff(k) = R.0
                } else {
                    k != n
                    not k < n
                    trichotomy(k, n)
                    if k < n {
                        false
                    }
                    if k = n {
                        false
                    }
                    n < k
                    lt_imp_lte_suc(n, k)
                    n.suc <= k
                    lte_imp_not_lt(n.suc, k)
                    not k < n.suc
                    coeff_zero_from_apply(p.coeff, n.suc, k)
                    p.coeff(k) = R.0
                }
                coeff_zero_at(p.coeff, k)
                coeff_zero_from_at(p.coeff, n, k) = (not k < n implies coeff_zero_at(p.coeff, k))
                coeff_zero_from_at(p.coeff, n, k)
            }
        }
        coeff_zero_from(p.coeff, n) = forall(k: Nat) { coeff_zero_from_at(p.coeff, n, k) }
        coeff_zero_from(p.coeff, n)
        polynomial_support_bounded_by(p, n) = coeff_zero_from(p.coeff, n)
        polynomial_support_bounded_by(p, n)
    }
}

/// One summand of the convolution coefficient at degree m + n + 1 vanishes
/// when both factors have the given degree bounds.
lemma polynomial_mul_term_coeff_zero_of_degree_le[R: Semiring](
    p: Polynomial[R], q: Polynomial[R], m: Nat, n: Nat, i: Nat
) {
    polynomial_degree_le(p, m) and polynomial_degree_le(q, n) and
    i < (m + n + Nat.1).suc implies
    polynomial_mul_term_coeff(p, q, m + n + Nat.1, i) = R.0
} by {
    if polynomial_degree_le(p, m) and polynomial_degree_le(q, n) and
        i < (m + n + Nat.1).suc {
        if i < m.suc {
            degree_nat_sub_gt(m.suc, n, i)
            not (m.suc + n - i) < n.suc
            m.suc + n = m + n + Nat.1
            not (m + n + Nat.1 - i) < n.suc
            polynomial_degree_le(q, n) = polynomial_support_bounded_by(q, n.suc)
            polynomial_support_bounded_by(q, n.suc)
            polynomial_support_bounded_by(q, n.suc) = coeff_zero_from(q.coeff, n.suc)
            coeff_zero_from(q.coeff, n.suc)
            coeff_zero_from_apply(q.coeff, n.suc, m + n + Nat.1 - i)
            q.coeff(m + n + Nat.1 - i) = R.0
            polynomial_mul_term_coeff(p, q, m + n + Nat.1, i) =
                p.coeff(i) * q.coeff(m + n + Nat.1 - i)
            polynomial_mul_term_coeff(p, q, m + n + Nat.1, i) = p.coeff(i) * R.0
            p.coeff(i) * R.0 = R.0
            polynomial_mul_term_coeff(p, q, m + n + Nat.1, i) = R.0
        } else {
            not i < m.suc
            polynomial_degree_le(p, m) = polynomial_support_bounded_by(p, m.suc)
            polynomial_support_bounded_by(p, m.suc)
            polynomial_support_bounded_by(p, m.suc) = coeff_zero_from(p.coeff, m.suc)
            coeff_zero_from(p.coeff, m.suc)
            coeff_zero_from_apply(p.coeff, m.suc, i)
            p.coeff(i) = R.0
            polynomial_mul_term_coeff(p, q, m + n + Nat.1, i) =
                p.coeff(i) * q.coeff(m + n + Nat.1 - i)
            polynomial_mul_term_coeff(p, q, m + n + Nat.1, i) = R.0 * q.coeff(m + n + Nat.1 - i)
            R.0 * q.coeff(m + n + Nat.1 - i) = R.0
            polynomial_mul_term_coeff(p, q, m + n + Nat.1, i) = R.0
        }
    }
}

/// The sharp degree bound for a product: the degree of a product is at most
/// the sum of the degrees.
theorem polynomial_degree_mul_le_sum[R: Semiring](
    p: Polynomial[R], q: Polynomial[R], m: Nat, n: Nat
) {
    polynomial_degree_le(p, m) and polynomial_degree_le(q, n) implies
    polynomial_degree_le(polynomial_mul(p, q), m + n)
} by {
    if polynomial_degree_le(p, m) and polynomial_degree_le(q, n) {
        polynomial_degree_mul_le_add(p, q, m, n)
        polynomial_degree_le(polynomial_mul(p, q), m + n + Nat.1)
        polynomial_degree_le(polynomial_mul(p, q), m + n + Nat.1) =
            polynomial_support_bounded_by(polynomial_mul(p, q), (m + n + Nat.1).suc)
        polynomial_support_bounded_by(polynomial_mul(p, q), (m + n + Nat.1).suc)
        forall(i: Nat) {
            if i < (m + n + Nat.1).suc {
                polynomial_mul_term_coeff_zero_of_degree_le(p, q, m, n, i)
                polynomial_mul_term_coeff(p, q, m + n + Nat.1, i) = R.0
            }
        }
        range_sum_zero_of_vanishes(polynomial_mul_term_coeff(p, q, m + n + Nat.1), (m + n + Nat.1).suc)
        range_sum(polynomial_mul_term_coeff(p, q, m + n + Nat.1), (m + n + Nat.1).suc) = R.0
        polynomial_mul_coeff(p, q, m + n + Nat.1) =
            partial(function(i: Nat) { polynomial_mul_term_coeff(p, q, m + n + Nat.1, i) }, (m + n + Nat.1).suc)
        range_sum_eq_partial(polynomial_mul_term_coeff(p, q, m + n + Nat.1), (m + n + Nat.1).suc)
        range_sum(polynomial_mul_term_coeff(p, q, m + n + Nat.1), (m + n + Nat.1).suc) =
            partial(function(i: Nat) { polynomial_mul_term_coeff(p, q, m + n + Nat.1, i) }, (m + n + Nat.1).suc)
        polynomial_mul_coeff(p, q, m + n + Nat.1) = R.0
        polynomial_mul_coeff_apply(p, q, m + n + Nat.1)
        polynomial_mul(p, q).coeff(m + n + Nat.1) = R.0
        polynomial_degree_drop_top(polynomial_mul(p, q), m + n)
        polynomial_degree_le(polynomial_mul(p, q), m + n)
    }
}

/// Negating a polynomial preserves its degree bounds.
theorem polynomial_degree_neg_iff[R: Ring](p: Polynomial[R], d: Nat) {
    polynomial_degree_le(p.neg, d) = polynomial_degree_le(p, d)
} by {
    if polynomial_degree_le(p.neg, d) {
        polynomial_degree_le(p.neg, d) = polynomial_support_bounded_by(p.neg, d.suc)
        polynomial_support_bounded_by(p.neg, d.suc)
        polynomial_support_bounded_by(p.neg, d.suc) = coeff_zero_from(p.neg.coeff, d.suc)
        coeff_zero_from(p.neg.coeff, d.suc)
        forall(k: Nat) {
            if not k < d.suc {
                coeff_zero_from_apply(p.neg.coeff, d.suc, k)
                p.neg.coeff(k) = R.0
                polynomial_neg_coeff(p, k)
                p.neg.coeff(k) = -p.coeff(k)
                -p.coeff(k) = R.0
                inverse_inverse(p.coeff(k))
                p.coeff(k) = -(-p.coeff(k))
                p.coeff(k) = -R.0
                inverse_left(R.0)
                -R.0 + R.0 = R.0
                R.0 + -R.0 = R.0
                R.0 + -R.0 = -R.0
                -R.0 = R.0
                p.coeff(k) = R.0
                coeff_zero_at(p.coeff, k)
                coeff_zero_from_at(p.coeff, d.suc, k) = (not k < d.suc implies coeff_zero_at(p.coeff, k))
                coeff_zero_from_at(p.coeff, d.suc, k)
            }
        }
        coeff_zero_from(p.coeff, d.suc) = forall(k: Nat) { coeff_zero_from_at(p.coeff, d.suc, k) }
        coeff_zero_from(p.coeff, d.suc)
        polynomial_support_bounded_by(p, d.suc) = coeff_zero_from(p.coeff, d.suc)
        polynomial_support_bounded_by(p, d.suc)
        polynomial_degree_le(p, d) = polynomial_support_bounded_by(p, d.suc)
        polynomial_degree_le(p, d)
    }
    if polynomial_degree_le(p, d) {
        polynomial_degree_le(p, d) = polynomial_support_bounded_by(p, d.suc)
        polynomial_support_bounded_by(p, d.suc)
        polynomial_support_bounded_by(p, d.suc) = coeff_zero_from(p.coeff, d.suc)
        coeff_zero_from(p.coeff, d.suc)
        forall(k: Nat) {
            if not k < d.suc {
                coeff_zero_from_apply(p.coeff, d.suc, k)
                p.coeff(k) = R.0
                polynomial_neg_coeff(p, k)
                p.neg.coeff(k) = -p.coeff(k)
                p.neg.coeff(k) = -R.0
                inverse_left(R.0)
                -R.0 + R.0 = R.0
                R.0 + -R.0 = R.0
                R.0 + -R.0 = -R.0
                -R.0 = R.0
                p.neg.coeff(k) = R.0
                coeff_zero_at(p.neg.coeff, k)
                coeff_zero_from_at(p.neg.coeff, d.suc, k) = (not k < d.suc implies coeff_zero_at(p.neg.coeff, k))
                coeff_zero_from_at(p.neg.coeff, d.suc, k)
            }
        }
        coeff_zero_from(p.neg.coeff, d.suc) =
            forall(k: Nat) { coeff_zero_from_at(p.neg.coeff, d.suc, k) }
        coeff_zero_from(p.neg.coeff, d.suc)
        polynomial_support_bounded_by(p.neg, d.suc) = coeff_zero_from(p.neg.coeff, d.suc)
        polynomial_support_bounded_by(p.neg, d.suc)
        polynomial_degree_le(p.neg, d) = polynomial_support_bounded_by(p.neg, d.suc)
        polynomial_degree_le(p.neg, d)
    }
    polynomial_degree_le(p.neg, d) = polynomial_degree_le(p, d)
}

/// The classical bound for the degree of a difference: the degree of a
/// difference is at most the maximum of the degrees.
theorem polynomial_degree_sub_le_max[R: Ring](
    p: Polynomial[R], q: Polynomial[R], m: Nat, n: Nat
) {
    polynomial_degree_le(p, m) and polynomial_degree_le(q, n) implies
    polynomial_degree_le(p.sub(q), m.max(n))
} by {
    if polynomial_degree_le(p, m) and polynomial_degree_le(q, n) {
        polynomial_degree_neg_iff(q, n)
        polynomial_degree_le(q.neg, n) = polynomial_degree_le(q, n)
        polynomial_degree_le(q.neg, n)
        polynomial_degree_sum_le_max(p, q.neg, m, n)
        polynomial_degree_le(p + q.neg, m.max(n))
        p.sub(q) = p + q.neg
        polynomial_degree_le(p.sub(q), m.max(n))
    }
}
