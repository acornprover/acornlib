/// Concrete polynomial theorems over the real numbers: the factor theorem
/// for `Real`, and the factorization and roots of `X^2 - 1`.

from nat import Nat
from algebra.ring.ring import mul_neg_right
from algebra.add_group import inverse_left, inverse_inverse
from order import lt_imp_lte
from nat import lt_suc, lte_trans
from real import Real
from polynomial import Polynomial, polynomial_constant, polynomial_monomial, polynomial_eval,
    polynomial_mul, polynomial_divisible_by_x_minus_a_polynomial,
    polynomial_factor_theorem_forward_global, polynomial_factor_theorem_global,
    polynomial_support_bounded_by, polynomial_support_bounded_by_monotone,
    polynomial_add_support_bounded_by, polynomial_constant_support_bounded_by_one
from polynomial.polynomial_vieta import linear_polynomial, quadratic_polynomial,
    quadratic_root_vieta_a, quadratic_root_vieta_b, polynomial_linear_vieta_divides,
    quadratic_eval_factorized, quadratic_polynomial_coeff_two, polynomial_linear_mul_quadratic
from polynomial.polynomial_eval_deep import polynomial_monomial_support_bounded_by_suc
from polynomial.polynomial_degree import polynomial_degree_le,
    polynomial_coeff_nonzero_not_degree_lt

numerals Real

/// The polynomial `X^2 - 1` over the reals.
let real_x_squared_minus_one: Polynomial[Real] =
    quadratic_polynomial(Real.1, Real.0, -Real.1)

/// The factor theorem over the reals: `a` is a root of `p` exactly when
/// `X - a` divides `p`.
theorem polynomial_real_factor_theorem(p: Polynomial[Real], a: Real) {
    (polynomial_eval(p, a) = Real.0) =
        polynomial_divisible_by_x_minus_a_polynomial(p, a)
} by {
    polynomial_factor_theorem_global(p, a)
}

/// The forward direction of the factor theorem over the reals.
theorem polynomial_real_factor_theorem_forward(p: Polynomial[Real], a: Real) {
    polynomial_eval(p, a) = Real.0 implies
    polynomial_divisible_by_x_minus_a_polynomial(p, a)
} by {
    if polynomial_eval(p, a) = Real.0 {
        polynomial_factor_theorem_forward_global(p, a)
        polynomial_divisible_by_x_minus_a_polynomial(p, a)
    }
}

/// The polynomial `X^2 - 1` is the Vieta quadratic with roots `1` and `-1`.
theorem real_x_squared_minus_one_eq_vieta {
    real_x_squared_minus_one =
        quadratic_polynomial(Real.1, -(Real.1 + -Real.1), Real.1 * -Real.1)
} by {
    inverse_left(Real.1)
    Real.1 + -Real.1 = Real.0
    -(Real.1 + -Real.1) = -Real.0
    -Real.0 = Real.0
    -(Real.1 + -Real.1) = Real.0
    mul_neg_right(Real.1, Real.1)
    Real.1 * -Real.1 = -(Real.1 * Real.1)
    Real.1 * Real.1 = Real.1
    Real.1 * -Real.1 = -Real.1
    quadratic_polynomial(Real.1, Real.0, -Real.1) =
        quadratic_polynomial(Real.1, -(Real.1 + -Real.1), Real.1 * -Real.1)
    real_x_squared_minus_one =
        quadratic_polynomial(Real.1, -(Real.1 + -Real.1), Real.1 * -Real.1)
}

/// `1` is a root of `X^2 - 1`.
theorem real_x_squared_minus_one_root_one {
    polynomial_eval(real_x_squared_minus_one, Real.1) = Real.0
} by {
    real_x_squared_minus_one_eq_vieta
    real_x_squared_minus_one =
        quadratic_polynomial(Real.1, -(Real.1 + -Real.1), Real.1 * -Real.1)
    polynomial_eval(real_x_squared_minus_one, Real.1) =
        polynomial_eval(quadratic_polynomial(Real.1, -(Real.1 + -Real.1), Real.1 * -Real.1), Real.1)
    quadratic_root_vieta_a(Real.1, -Real.1)
    polynomial_eval(quadratic_polynomial(Real.1, -(Real.1 + -Real.1), Real.1 * -Real.1), Real.1) =
        Real.0
    polynomial_eval(real_x_squared_minus_one, Real.1) = Real.0
}

/// `-1` is a root of `X^2 - 1`.
theorem real_x_squared_minus_one_root_neg_one {
    polynomial_eval(real_x_squared_minus_one, -Real.1) = Real.0
} by {
    real_x_squared_minus_one_eq_vieta
    real_x_squared_minus_one =
        quadratic_polynomial(Real.1, -(Real.1 + -Real.1), Real.1 * -Real.1)
    polynomial_eval(real_x_squared_minus_one, -Real.1) =
        polynomial_eval(quadratic_polynomial(Real.1, -(Real.1 + -Real.1), Real.1 * -Real.1), -Real.1)
    quadratic_root_vieta_b(Real.1, -Real.1)
    polynomial_eval(quadratic_polynomial(Real.1, -(Real.1 + -Real.1), Real.1 * -Real.1), -Real.1) =
        Real.0
    polynomial_eval(real_x_squared_minus_one, -Real.1) = Real.0
}

/// `X - 1` divides `X^2 - 1` over the reals.
theorem real_x_minus_one_divides_x_squared_minus_one {
    polynomial_divisible_by_x_minus_a_polynomial(real_x_squared_minus_one, Real.1)
} by {
    real_x_squared_minus_one_eq_vieta
    real_x_squared_minus_one =
        quadratic_polynomial(Real.1, -(Real.1 + -Real.1), Real.1 * -Real.1)
    polynomial_linear_vieta_divides(Real.1, -Real.1)
    polynomial_divisible_by_x_minus_a_polynomial(
        quadratic_polynomial(Real.1, -(Real.1 + -Real.1), Real.1 * -Real.1), Real.1)
    polynomial_divisible_by_x_minus_a_polynomial(real_x_squared_minus_one, Real.1)
}

/// The concrete factor theorem: `X^2 - 1` has `1` as a root exactly when
/// `X - 1` divides it.
theorem real_x_squared_minus_one_root_iff_divides {
    (polynomial_eval(real_x_squared_minus_one, Real.1) = Real.0) =
        polynomial_divisible_by_x_minus_a_polynomial(real_x_squared_minus_one, Real.1)
} by {
    polynomial_real_factor_theorem(real_x_squared_minus_one, Real.1)
}

/// `X - 1` divides `X^2 - 1` over the reals, witnessed by `X + 1` at every
/// point: the factorized evaluation.
theorem real_x_squared_minus_one_eval_factorized(x: Real) {
    polynomial_eval(real_x_squared_minus_one, x) =
        polynomial_eval(linear_polynomial(Real.1, Real.1), x) * (x - Real.1)
} by {
    real_x_squared_minus_one_eq_vieta
    real_x_squared_minus_one =
        quadratic_polynomial(Real.1, -(Real.1 + -Real.1), Real.1 * -Real.1)
    quadratic_eval_factorized(Real.1, -Real.1, x)
    polynomial_eval(quadratic_polynomial(Real.1, -(Real.1 + -Real.1), Real.1 * -Real.1), x) =
        polynomial_eval(linear_polynomial(Real.1, -(-Real.1)), x) * (x - Real.1)
    -(-Real.1) = Real.1
    linear_polynomial(Real.1, -(-Real.1)) = linear_polynomial(Real.1, Real.1)
    polynomial_eval(linear_polynomial(Real.1, -(-Real.1)), x) =
        polynomial_eval(linear_polynomial(Real.1, Real.1), x)
    polynomial_eval(real_x_squared_minus_one, x) =
        polynomial_eval(linear_polynomial(Real.1, Real.1), x) * (x - Real.1)
}

/// The polynomial `X^2 - X` over the reals.
let real_x_squared_minus_x: Polynomial[Real] =
    quadratic_polynomial(Real.1, -Real.1, Real.0)

/// `X^2 - X` is the Vieta quadratic with roots `0` and `1`.
theorem real_x_squared_minus_x_eq_vieta {
    real_x_squared_minus_x =
        quadratic_polynomial(Real.1, -(Real.0 + Real.1), Real.0 * Real.1)
} by {
    Real.0 + Real.1 = Real.1
    -(Real.0 + Real.1) = -Real.1
    Real.0 * Real.1 = Real.0
    quadratic_polynomial(Real.1, -Real.1, Real.0) =
        quadratic_polynomial(Real.1, -(Real.0 + Real.1), Real.0 * Real.1)
    real_x_squared_minus_x =
        quadratic_polynomial(Real.1, -(Real.0 + Real.1), Real.0 * Real.1)
}

/// `0` is a root of `X^2 - X`.
theorem real_x_squared_minus_x_root_zero {
    polynomial_eval(real_x_squared_minus_x, Real.0) = Real.0
} by {
    real_x_squared_minus_x_eq_vieta
    real_x_squared_minus_x =
        quadratic_polynomial(Real.1, -(Real.0 + Real.1), Real.0 * Real.1)
    polynomial_eval(real_x_squared_minus_x, Real.0) =
        polynomial_eval(quadratic_polynomial(Real.1, -(Real.0 + Real.1), Real.0 * Real.1), Real.0)
    quadratic_root_vieta_a(Real.0, Real.1)
    polynomial_eval(quadratic_polynomial(Real.1, -(Real.0 + Real.1), Real.0 * Real.1), Real.0) =
        Real.0
    polynomial_eval(real_x_squared_minus_x, Real.0) = Real.0
}

/// `1` is a root of `X^2 - X`.
theorem real_x_squared_minus_x_root_one {
    polynomial_eval(real_x_squared_minus_x, Real.1) = Real.0
} by {
    real_x_squared_minus_x_eq_vieta
    real_x_squared_minus_x =
        quadratic_polynomial(Real.1, -(Real.0 + Real.1), Real.0 * Real.1)
    polynomial_eval(real_x_squared_minus_x, Real.1) =
        polynomial_eval(quadratic_polynomial(Real.1, -(Real.0 + Real.1), Real.0 * Real.1), Real.1)
    quadratic_root_vieta_b(Real.0, Real.1)
    polynomial_eval(quadratic_polynomial(Real.1, -(Real.0 + Real.1), Real.0 * Real.1), Real.1) =
        Real.0
    polynomial_eval(real_x_squared_minus_x, Real.1) = Real.0
}

/// The factorization `X^2 - 1 = (X - 1)(X + 1)` over the reals.
theorem real_x_squared_minus_one_eq_mul {
    polynomial_mul(linear_polynomial(Real.1, -Real.1), linear_polynomial(Real.1, Real.1)) =
        real_x_squared_minus_one
} by {
    polynomial_linear_mul_quadratic(Real.1, -Real.1)
    polynomial_mul(linear_polynomial(Real.1, -Real.1), linear_polynomial(Real.1, -(-Real.1))) =
        quadratic_polynomial(Real.1, -(Real.1 + -Real.1), Real.1 * -Real.1)
    -(-Real.1) = Real.1
    linear_polynomial(Real.1, -(-Real.1)) = linear_polynomial(Real.1, Real.1)
    polynomial_mul(linear_polynomial(Real.1, -Real.1), linear_polynomial(Real.1, Real.1)) =
        quadratic_polynomial(Real.1, -(Real.1 + -Real.1), Real.1 * -Real.1)
    inverse_left(Real.1)
    Real.1 + -Real.1 = Real.0
    -(Real.1 + -Real.1) = -Real.0
    -Real.0 = Real.0
    mul_neg_right(Real.1, Real.1)
    Real.1 * -Real.1 = -(Real.1 * Real.1)
    Real.1 * Real.1 = Real.1
    Real.1 * -Real.1 = -Real.1
    quadratic_polynomial(Real.1, -(Real.1 + -Real.1), Real.1 * -Real.1) =
        quadratic_polynomial(Real.1, Real.0, -Real.1)
    polynomial_mul(linear_polynomial(Real.1, -Real.1), linear_polynomial(Real.1, Real.1)) =
        quadratic_polynomial(Real.1, Real.0, -Real.1)
    polynomial_mul(linear_polynomial(Real.1, -Real.1), linear_polynomial(Real.1, Real.1)) =
        real_x_squared_minus_one
}

/// Two is at most three.
lemma real_nat_two_le_three {
    Nat.2 <= Nat.3
} by {
    lt_suc(Nat.2)
    Nat.2 < Nat.2.suc
    Nat.2.suc = Nat.3
    lt_imp_lte[Nat](Nat.2, Nat.3)
    Nat.2 <= Nat.3
}

/// One is at most three.
lemma real_nat_one_le_three {
    Nat.1 <= Nat.3
} by {
    lt_suc(Nat.1)
    Nat.1 < Nat.1.suc
    Nat.1.suc = Nat.2
    lt_imp_lte[Nat](Nat.1, Nat.2)
    Nat.1 <= Nat.2
    lt_suc(Nat.2)
    Nat.2 < Nat.2.suc
    Nat.2.suc = Nat.3
    lt_imp_lte[Nat](Nat.2, Nat.3)
    Nat.2 <= Nat.3
    lte_trans(Nat.1, Nat.2, Nat.3)
    Nat.1 <= Nat.3
}

/// The polynomial `X^2 - 1` has degree exactly two over the reals.
theorem real_x_squared_minus_one_degree {
    polynomial_degree_le(real_x_squared_minus_one, Nat.2) and
    not polynomial_degree_le(real_x_squared_minus_one, Nat.1)
} by {
    polynomial_constant_support_bounded_by_one(-Real.1)
    polynomial_support_bounded_by(polynomial_constant(-Real.1), Nat.1)
    real_nat_one_le_three
    polynomial_support_bounded_by_monotone(polynomial_constant(-Real.1), Nat.1, Nat.3)
    polynomial_support_bounded_by(polynomial_constant(-Real.1), Nat.3)
    polynomial_monomial_support_bounded_by_suc(Nat.1, Real.0)
    polynomial_support_bounded_by(polynomial_monomial(Nat.1, Real.0), Nat.2)
    real_nat_two_le_three
    polynomial_support_bounded_by_monotone(polynomial_monomial(Nat.1, Real.0), Nat.2, Nat.3)
    polynomial_support_bounded_by(polynomial_monomial(Nat.1, Real.0), Nat.3)
    polynomial_add_support_bounded_by(polynomial_constant(-Real.1), polynomial_monomial(Nat.1, Real.0), Nat.3)
    polynomial_support_bounded_by(polynomial_constant(-Real.1) + polynomial_monomial(Nat.1, Real.0), Nat.3)
    polynomial_monomial_support_bounded_by_suc(Nat.2, Real.1)
    polynomial_support_bounded_by(polynomial_monomial(Nat.2, Real.1), Nat.3)
    polynomial_add_support_bounded_by(polynomial_constant(-Real.1) + polynomial_monomial(Nat.1, Real.0),
        polynomial_monomial(Nat.2, Real.1), Nat.3)
    polynomial_support_bounded_by(
        (polynomial_constant(-Real.1) + polynomial_monomial(Nat.1, Real.0)) +
        polynomial_monomial(Nat.2, Real.1), Nat.3)
    real_x_squared_minus_one =
        (polynomial_constant(-Real.1) + polynomial_monomial(Nat.1, Real.0)) +
        polynomial_monomial(Nat.2, Real.1)
    polynomial_support_bounded_by(real_x_squared_minus_one, Nat.3)
    polynomial_degree_le(real_x_squared_minus_one, Nat.2) =
        polynomial_support_bounded_by(real_x_squared_minus_one, Nat.2.suc)
    polynomial_degree_le(real_x_squared_minus_one, Nat.2)
    quadratic_polynomial_coeff_two(Real.1, Real.0, -Real.1)
    real_x_squared_minus_one.coeff(Nat.2) = Real.1
    Real.1 != Real.0
    real_x_squared_minus_one.coeff(Nat.2) != Real.0
    polynomial_coeff_nonzero_not_degree_lt(real_x_squared_minus_one, Nat.2, Nat.1)
    not (polynomial_degree_le(real_x_squared_minus_one, Nat.1) and Nat.1 < Nat.2)
    if polynomial_degree_le(real_x_squared_minus_one, Nat.1) {
        false
    }
    not polynomial_degree_le(real_x_squared_minus_one, Nat.1)
    polynomial_degree_le(real_x_squared_minus_one, Nat.2) and
        not polynomial_degree_le(real_x_squared_minus_one, Nat.1)
}
