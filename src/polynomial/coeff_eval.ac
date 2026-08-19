/// Bounded coefficient-polynomial evaluation and linear-factor theorems.

from nat import Nat
from nat import alt_induction
from data.basic.function_algebra import pointwise_add
from data.basic.functions import function_extensionality
from semiring import Semiring
from comm_ring import CommRing
from algebra.comm_semigroup import CommSemigroup
from algebra.semigroup import Semigroup

/// The coefficient function obtained by dropping the constant coefficient.
define coeff_tail[R](c: Nat -> R, i: Nat) -> R {
    c(i.suc)
}

/// Horner evaluation of the polynomial determined by the first `n` coefficients.
define coeff_eval[R: Semiring](c: Nat -> R, x: R, n: Nat) -> R {
    match n {
        Nat.zero {
            R.0
        }
        Nat.suc(pred) {
            c(Nat.0) + x * coeff_eval(coeff_tail(c), x, pred)
        }
    }
}

/// The synthetic quotient coefficients for division by `X - a`, using the first `n` coefficients.
define coeff_quotient[R: Semiring](c: Nat -> R, a: R, n: Nat, i: Nat) -> R {
    match n {
        Nat.zero {
            R.0
        }
        Nat.suc(pred) {
            match i {
                Nat.zero {
                    coeff_eval(coeff_tail(c), a, pred)
                }
                Nat.suc(ipred) {
                    coeff_quotient(coeff_tail(c), a, pred, ipred)
                }
            }
        }
    }
}

/// The value at `x` of a quotient polynomial multiplied by the linear factor `X - a`.
define coeff_mul_x_minus_a[R: CommRing](q: Nat -> R, a: R, x: R, n: Nat) -> R {
    coeff_eval(q, x, n) * (x - a)
}


/// The tail of a pointwise sum is the pointwise sum of tails.
lemma coeff_tail_pointwise_add[R: Semiring](c: Nat -> R, d: Nat -> R) {
    coeff_tail(pointwise_add(c, d)) = pointwise_add(coeff_tail(c), coeff_tail(d))
} by {
    forall(i: Nat) {
        coeff_tail(pointwise_add(c, d), i) = pointwise_add(c, d, i.suc)
        pointwise_add(c, d, i.suc) = c(i.suc) + d(i.suc)
        coeff_tail(c, i) = c(i.suc)
        coeff_tail(d, i) = d(i.suc)
        pointwise_add(coeff_tail(c), coeff_tail(d), i) = coeff_tail(c, i) + coeff_tail(d, i)
        coeff_tail(pointwise_add(c, d), i) = pointwise_add(coeff_tail(c), coeff_tail(d), i)
    }
    function_extensionality(coeff_tail(pointwise_add(c, d)), pointwise_add(coeff_tail(c), coeff_tail(d)))
}

lemma coeff_add_pair_rearrange[R: Semiring](a: R, b: R, c: R, d: R) {
    a + b + (c + d) = (a + c) + (b + d)
} by {
    a + b + (c + d) = a + (b + (c + d))
    b + (c + d) = b + c + d
    b + c = c + b
    b + c + d = c + b + d
    c + b + d = c + (b + d)
    a + (b + (c + d)) = a + (c + (b + d))
    a + (c + (b + d)) = a + c + (b + d)
    a + b + (c + d) = (a + c) + (b + d)
}

define coeff_eval_add_at[R: Semiring](c: Nat -> R, d: Nat -> R, x: R, n: Nat) -> Bool {
    coeff_eval(pointwise_add(c, d), x, n) = coeff_eval(c, x, n) + coeff_eval(d, x, n)
}

define coeff_eval_add_property[R: Semiring](x: R, n: Nat) -> Bool {
    forall(c: Nat -> R, d: Nat -> R) {
        coeff_eval_add_at(c, d, x, n)
    }
}

lemma coeff_eval_add_property_apply[R: Semiring](x: R, n: Nat, c: Nat -> R, d: Nat -> R) {
    coeff_eval_add_property(x, n) implies coeff_eval_add_at(c, d, x, n)
} by {
    if coeff_eval_add_property(x, n) {
        coeff_eval_add_property(x, n) = forall(e: Nat -> R, f: Nat -> R) {
            coeff_eval_add_at(e, f, x, n)
        }
        coeff_eval_add_at(c, d, x, n)
    }
}

lemma coeff_eval_add_property_intro[R: Semiring](x: R, n: Nat) {
    (forall(c: Nat -> R, d: Nat -> R) { coeff_eval_add_at(c, d, x, n) }) implies
    coeff_eval_add_property(x, n)
} by {
    forall(c: Nat -> R, d: Nat -> R) {
        coeff_eval_add_at(c, d, x, n)
    }
    coeff_eval_add_property(x, n) = forall(c: Nat -> R, d: Nat -> R) {
        coeff_eval_add_at(c, d, x, n)
    }
}

lemma coeff_eval_add_property_all[R: Semiring](x: R, n: Nat) {
    coeff_eval_add_property(x, n)
} by {
    define statement(k: Nat) -> Bool {
        coeff_eval_add_property(x, k)
    }

    forall(c: Nat -> R, d: Nat -> R) {
        coeff_eval(pointwise_add(c, d), x, Nat.0) = R.0
        coeff_eval(c, x, Nat.0) = R.0
        coeff_eval(d, x, Nat.0) = R.0
        coeff_eval(c, x, Nat.0) + coeff_eval(d, x, Nat.0) = R.0 + R.0
        R.0 + R.0 = R.0
        coeff_eval(pointwise_add(c, d), x, Nat.0) = coeff_eval(c, x, Nat.0) + coeff_eval(d, x, Nat.0)
        coeff_eval_add_at(c, d, x, Nat.0) =
            (coeff_eval(pointwise_add(c, d), x, Nat.0) = coeff_eval(c, x, Nat.0) + coeff_eval(d, x, Nat.0))
        coeff_eval_add_at(c, d, x, Nat.0)
    }
    coeff_eval_add_property_intro(x, Nat.0)
    coeff_eval_add_property(x, Nat.0)
    statement(Nat.0)

    forall(k: Nat) {
        if statement(k) {
            forall(c: Nat -> R, d: Nat -> R) {
                let ct = coeff_tail(c)
                let dt = coeff_tail(d)
                let s = pointwise_add(c, d)
                ct = coeff_tail(c)
                dt = coeff_tail(d)
                s = pointwise_add(c, d)
                coeff_tail_pointwise_add(c, d)
                coeff_tail(s) = pointwise_add(ct, dt)
                statement(k) = coeff_eval_add_property(x, k)
                coeff_eval_add_property_apply(x, k, ct, dt)
                coeff_eval_add_at(ct, dt, x, k)
                coeff_eval(pointwise_add(ct, dt), x, k) = coeff_eval(ct, x, k) + coeff_eval(dt, x, k)

                coeff_eval(s, x, k.suc) = s(Nat.0) + x * coeff_eval(coeff_tail(s), x, k)
                s(Nat.0) = c(Nat.0) + d(Nat.0)
                coeff_eval(s, x, k.suc) =
                    c(Nat.0) + d(Nat.0) + x * coeff_eval(pointwise_add(ct, dt), x, k)
                coeff_eval(s, x, k.suc) =
                    c(Nat.0) + d(Nat.0) + x * (coeff_eval(ct, x, k) + coeff_eval(dt, x, k))
                x * (coeff_eval(ct, x, k) + coeff_eval(dt, x, k)) =
                    x * coeff_eval(ct, x, k) + x * coeff_eval(dt, x, k)
                coeff_eval(s, x, k.suc) =
                    c(Nat.0) + d(Nat.0) + (x * coeff_eval(ct, x, k) + x * coeff_eval(dt, x, k))

                ct = coeff_tail(c)
                dt = coeff_tail(d)
                coeff_eval(c, x, k.suc) = c(Nat.0) + x * coeff_eval(coeff_tail(c), x, k)
                coeff_eval(c, x, k.suc) = c(Nat.0) + x * coeff_eval(ct, x, k)
                coeff_eval(d, x, k.suc) = d(Nat.0) + x * coeff_eval(coeff_tail(d), x, k)
                coeff_eval(d, x, k.suc) = d(Nat.0) + x * coeff_eval(dt, x, k)
                coeff_add_pair_rearrange(c(Nat.0), d(Nat.0), x * coeff_eval(ct, x, k), x * coeff_eval(dt, x, k))
                c(Nat.0) + d(Nat.0) + (x * coeff_eval(ct, x, k) + x * coeff_eval(dt, x, k)) =
                    (c(Nat.0) + x * coeff_eval(ct, x, k)) + (d(Nat.0) + x * coeff_eval(dt, x, k))
                coeff_eval(s, x, k.suc) = coeff_eval(c, x, k.suc) + coeff_eval(d, x, k.suc)
                coeff_eval_add_at(c, d, x, k.suc) =
                    (coeff_eval(pointwise_add(c, d), x, k.suc) = coeff_eval(c, x, k.suc) + coeff_eval(d, x, k.suc))
                s = pointwise_add(c, d)
                coeff_eval(pointwise_add(c, d), x, k.suc) = coeff_eval(c, x, k.suc) + coeff_eval(d, x, k.suc)
                coeff_eval_add_at(c, d, x, k.suc)
            }
            coeff_eval_add_property_intro(x, k.suc)
            coeff_eval_add_property(x, k.suc)
            statement(k.suc)
        }
    }
    alt_induction(statement)
    forall(m: Nat) { statement(m) }
    statement(n)
}

/// Evaluating a coefficient sequence at bound zero yields the zero element.
theorem coeff_eval_zero_index[R: Semiring](c: Nat -> R, x: R) {
    coeff_eval(c, x, Nat.0) = R.0
} by {
    coeff_eval(c, x, Nat.0) = R.0
}

theorem coeff_eval_add[R: Semiring](c: Nat -> R, d: Nat -> R, x: R, n: Nat) {
    coeff_eval(pointwise_add(c, d), x, n) = coeff_eval(c, x, n) + coeff_eval(d, x, n)
} by {
    coeff_eval_add_property_all(x, n)
    coeff_eval_add_property_apply(x, n, c, d)
    coeff_eval_add_at(c, d, x, n)
}

/// A bounded coefficient polynomial is divisible by `X - a` when it is represented by such a product.
define coeff_divisible_by_x_minus_a[R: CommRing](c: Nat -> R, a: R, n: Nat) -> Bool {
    exists(q: Nat -> R) {
        forall(x: R) {
            coeff_eval(c, x, n) = coeff_mul_x_minus_a(q, a, x, n)
        }
    }
}

/// The constant coefficient of the synthetic quotient is the tail evaluated at the root.
theorem coeff_quotient_zero_index[R: Semiring](c: Nat -> R, a: R, n: Nat) {
    coeff_quotient(c, a, n.suc, Nat.0) = coeff_eval(coeff_tail(c), a, n)
}

/// Adding the displacement from `a` to `x` to `a` gives `x` in the linear-factor proof.
lemma linear_factor_add_sub_cancel_left[R: CommRing](a: R, x: R) {
    a + (x - a) = x
} by {
    x - a = x + -a
    a + (x - a) = a + (x + -a)
    a + (x + -a) = a + (-a + x)
    a + (-a + x) = (a + -a) + x
    a + -a = R.0
    (a + -a) + x = R.0 + x
    R.0 + x = x
}

/// Multiplication by `x` splits through an intermediate scalar `a` in the linear-factor proof.
lemma linear_factor_mul_sub_split[R: CommRing](a: R, x: R, t: R) {
    x * t = a * t + (x - a) * t
} by {
    linear_factor_add_sub_cancel_left(a, x)
    x = a + (x - a)
    x * t = (a + (x - a)) * t
    (a + (x - a)) * t = a * t + (x - a) * t
}

/// Two adjacent factors may be swapped inside a triple product in the linear-factor proof.
lemma linear_factor_mul_assoc_swap_middle[R: CommRing](x: R, y: R, z: R) {
    x * (y * z) = y * (x * z)
} by {
    Semigroup.mul_associative[R](x, y, z)
    x * (y * z) = (x * y) * z
    CommSemigroup.commutative[R](x, y)
    x * y = y * x
    (x * y) * z = (y * x) * z
    Semigroup.mul_associative[R](y, x, z)
    y * (x * z) = (y * x) * z
    x * (y * z) = y * (x * z)
}

/// The tail of the synthetic quotient is the synthetic quotient of the tail.
theorem coeff_tail_quotient[R: Semiring](c: Nat -> R, a: R, n: Nat) {
    coeff_tail(coeff_quotient(c, a, n.suc), Nat.0) = coeff_quotient(coeff_tail(c), a, n, Nat.0) and
    coeff_tail(coeff_quotient(c, a, n.suc)) = coeff_quotient(coeff_tail(c), a, n)
} by {
    coeff_tail(coeff_quotient(c, a, n.suc), Nat.0) = coeff_quotient(c, a, n.suc, Nat.1)
    coeff_quotient(c, a, n.suc, Nat.1) = coeff_quotient(coeff_tail(c), a, n, Nat.0)
    forall(i: Nat) {
        coeff_tail(coeff_quotient(c, a, n.suc), i) = coeff_quotient(c, a, n.suc, i.suc)
        coeff_quotient(c, a, n.suc, i.suc) = coeff_quotient(coeff_tail(c), a, n, i)
        coeff_tail(coeff_quotient(c, a, n.suc), i) = coeff_quotient(coeff_tail(c), a, n, i)
    }
}

/// The bounded remainder identity for a fixed coefficient function.
define coeff_remainder_at[R: CommRing](c: Nat -> R, a: R, x: R, n: Nat) -> Bool {
    coeff_eval(c, x, n) = coeff_eval(c, a, n) + (x - a) * coeff_eval(coeff_quotient(c, a, n), x, n)
}

/// The bounded remainder identity for all coefficient functions at a fixed bound.
define coeff_remainder_property[R: CommRing](a: R, x: R, n: Nat) -> Bool {
    forall(c: Nat -> R) {
        coeff_remainder_at(c, a, x, n)
    }
}

/// A universal bounded remainder property applies to any coefficient function.
lemma coeff_remainder_property_apply[R: CommRing](a: R, x: R, n: Nat, c: Nat -> R) {
    coeff_remainder_property(a, x, n) implies coeff_remainder_at(c, a, x, n)
} by {
    if coeff_remainder_property(a, x, n) {
        coeff_remainder_property(a, x, n) = forall(d: Nat -> R) {
            coeff_remainder_at(d, a, x, n)
        }
        coeff_remainder_at(c, a, x, n)
    }
}

/// A pointwise proof gives the universal bounded remainder property.
lemma coeff_remainder_property_intro[R: CommRing](a: R, x: R, n: Nat) {
    (forall(c: Nat -> R) { coeff_remainder_at(c, a, x, n) }) implies
    coeff_remainder_property(a, x, n)
} by {
    forall(c: Nat -> R) {
        coeff_remainder_at(c, a, x, n)
    }
    coeff_remainder_property(a, x, n) = forall(c: Nat -> R) {
        coeff_remainder_at(c, a, x, n)
    }
}

/// The bounded coefficient-form remainder theorem, uniformly in the coefficients.
lemma coeff_remainder_property_all[R: CommRing](a: R, x: R, n: Nat) {
    coeff_remainder_property(a, x, n)
} by {
    define statement(k: Nat) -> Bool {
        coeff_remainder_property(a, x, k)
    }

    forall(d: Nat -> R) {
        coeff_eval(d, x, Nat.0) = R.0
        coeff_eval(d, a, Nat.0) = R.0
        coeff_eval(coeff_quotient(d, a, Nat.0), x, Nat.0) = R.0
        R.0 + (x - a) * R.0 = R.0
        coeff_eval(d, x, Nat.0) =
            coeff_eval(d, a, Nat.0) + (x - a) * coeff_eval(coeff_quotient(d, a, Nat.0), x, Nat.0)
        coeff_remainder_at(d, a, x, Nat.0)
    }
    coeff_remainder_property_intro(a, x, Nat.0)
    coeff_remainder_property(a, x, Nat.0)
    statement(Nat.0)

    forall(k: Nat) {
        if statement(k) {
            forall(d: Nat -> R) {
                let dt = coeff_tail(d)
                let q = coeff_quotient(d, a, k.suc)
                let qt = coeff_quotient(dt, a, k)
                coeff_tail_quotient(d, a, k)
                coeff_tail(q) = qt
                q = coeff_quotient(d, a, k.suc)
                q(Nat.0) = coeff_quotient(d, a, k.suc, Nat.0)
                coeff_quotient_zero_index(d, a, k)
                coeff_quotient(d, a, k.suc, Nat.0) = coeff_eval(dt, a, k)
                q(Nat.0) = coeff_eval(dt, a, k)
                coeff_eval(q, x, k.suc) = q(Nat.0) + x * coeff_eval(coeff_tail(q), x, k)
                coeff_eval(q, x, k.suc) = coeff_eval(dt, a, k) + x * coeff_eval(qt, x, k)

                coeff_remainder_property_apply(a, x, k, dt)
                coeff_remainder_at(dt, a, x, k)
                coeff_eval(dt, x, k) = coeff_eval(dt, a, k) + (x - a) * coeff_eval(qt, x, k)

                coeff_eval(d, x, k.suc) = d(Nat.0) + x * coeff_eval(dt, x, k)
                coeff_eval(d, a, k.suc) = d(Nat.0) + a * coeff_eval(dt, a, k)
                coeff_eval(d, x, k.suc) = d(Nat.0) + x * (coeff_eval(dt, a, k) + (x - a) * coeff_eval(qt, x, k))
                d(Nat.0) + x * (coeff_eval(dt, a, k) + (x - a) * coeff_eval(qt, x, k)) =
                    d(Nat.0) + x * coeff_eval(dt, a, k) + x * ((x - a) * coeff_eval(qt, x, k))
                linear_factor_mul_assoc_swap_middle(x, x - a, coeff_eval(qt, x, k))
                x * ((x - a) * coeff_eval(qt, x, k)) = (x - a) * (x * coeff_eval(qt, x, k))
                d(Nat.0) + x * coeff_eval(dt, a, k) + x * ((x - a) * coeff_eval(qt, x, k)) =
                    d(Nat.0) + x * coeff_eval(dt, a, k) + (x - a) * (x * coeff_eval(qt, x, k))
                linear_factor_mul_sub_split(a, x, coeff_eval(dt, a, k))
                x * coeff_eval(dt, a, k) = a * coeff_eval(dt, a, k) + (x - a) * coeff_eval(dt, a, k)
                d(Nat.0) + x * coeff_eval(dt, a, k) =
                    d(Nat.0) + a * coeff_eval(dt, a, k) + (x - a) * coeff_eval(dt, a, k)
                d(Nat.0) + x * coeff_eval(dt, a, k) + (x - a) * (x * coeff_eval(qt, x, k)) =
                    d(Nat.0) + a * coeff_eval(dt, a, k) + (x - a) * coeff_eval(dt, a, k) +
                    (x - a) * (x * coeff_eval(qt, x, k))
                (x - a) * coeff_eval(dt, a, k) + (x - a) * (x * coeff_eval(qt, x, k)) =
                    (x - a) * (coeff_eval(dt, a, k) + x * coeff_eval(qt, x, k))
                d(Nat.0) + a * coeff_eval(dt, a, k) + (x - a) * coeff_eval(dt, a, k) +
                    (x - a) * (x * coeff_eval(qt, x, k)) =
                    d(Nat.0) + a * coeff_eval(dt, a, k) +
                    (x - a) * (coeff_eval(dt, a, k) + x * coeff_eval(qt, x, k))
                coeff_eval(d, x, k.suc) = coeff_eval(d, a, k.suc) + (x - a) * coeff_eval(q, x, k.suc)
                coeff_remainder_at(d, a, x, k.suc)
            }
            coeff_remainder_property_intro(a, x, k.suc)
            coeff_remainder_property(a, x, k.suc)
            statement(k.suc)
        }
    }

    alt_induction(statement)
    forall(m: Nat) { statement(m) }
    statement(n)
}

/// The bounded coefficient-form remainder theorem.
theorem coeff_remainder_theorem[R: CommRing](c: Nat -> R, a: R, x: R, n: Nat) {
    coeff_eval(c, x, n) = coeff_eval(c, a, n) + (x - a) * coeff_eval(coeff_quotient(c, a, n), x, n)
} by {
    coeff_remainder_property_all(a, x, n)
    coeff_remainder_property_apply(a, x, n, c)
    coeff_remainder_at(c, a, x, n)
}

/// A root gives divisibility by the corresponding linear factor.
theorem coeff_factor_theorem_forward[R: CommRing](c: Nat -> R, a: R, n: Nat) {
    coeff_eval(c, a, n) = R.0 implies coeff_divisible_by_x_minus_a(c, a, n)
} by {
    if coeff_eval(c, a, n) = R.0 {
        forall(x: R) {
            coeff_remainder_theorem(c, a, x, n)
            coeff_eval(c, x, n) = coeff_eval(c, a, n) + (x - a) * coeff_eval(coeff_quotient(c, a, n), x, n)
            coeff_eval(c, a, n) = R.0
            R.0 + (x - a) * coeff_eval(coeff_quotient(c, a, n), x, n) =
                (x - a) * coeff_eval(coeff_quotient(c, a, n), x, n)
            (x - a) * coeff_eval(coeff_quotient(c, a, n), x, n) =
                coeff_eval(coeff_quotient(c, a, n), x, n) * (x - a)
            coeff_mul_x_minus_a(coeff_quotient(c, a, n), a, x, n) =
                coeff_eval(coeff_quotient(c, a, n), x, n) * (x - a)
            coeff_eval(c, x, n) = coeff_mul_x_minus_a(coeff_quotient(c, a, n), a, x, n)
        }
        exists(q: Nat -> R) {
            forall(x: R) {
                coeff_eval(c, x, n) = coeff_mul_x_minus_a(q, a, x, n)
            }
        }
        coeff_divisible_by_x_minus_a(c, a, n)
    }
}

/// Divisibility by `X - a` forces `a` to be a root.
theorem coeff_factor_theorem_reverse[R: CommRing](c: Nat -> R, a: R, n: Nat) {
    coeff_divisible_by_x_minus_a(c, a, n) implies coeff_eval(c, a, n) = R.0
} by {
    if coeff_divisible_by_x_minus_a(c, a, n) {
        let q: Nat -> R satisfy {
            forall(x: R) {
                coeff_eval(c, x, n) = coeff_mul_x_minus_a(q, a, x, n)
            }
        }
        coeff_eval(c, a, n) = coeff_mul_x_minus_a(q, a, a, n)
        coeff_mul_x_minus_a(q, a, a, n) = coeff_eval(q, a, n) * (a - a)
        a - a = a + -a
        a + -a = R.0
        a - a = R.0
        coeff_eval(q, a, n) * R.0 = R.0
        coeff_eval(c, a, n) = R.0
    }
}

/// The bounded coefficient-form factor theorem for commutative rings.
theorem coeff_factor_theorem[R: CommRing](c: Nat -> R, a: R, n: Nat) {
    coeff_eval(c, a, n) = R.0 = coeff_divisible_by_x_minus_a(c, a, n)
} by {
    if coeff_eval(c, a, n) = R.0 {
        coeff_factor_theorem_forward(c, a, n)
        coeff_divisible_by_x_minus_a(c, a, n)
    }
    if coeff_divisible_by_x_minus_a(c, a, n) {
        coeff_factor_theorem_reverse(c, a, n)
        coeff_eval(c, a, n) = R.0
    }
}
