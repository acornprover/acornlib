/// Global factor theorem and global divisibility for polynomial evaluation.

from comm_ring import CommRing
from nat import Nat
from polynomial.base import Polynomial
from polynomial.eval import polynomial_support_bounded_by, polynomial_eval_bound,
    polynomial_quotient, polynomial_quotient_support_bounded_by,
    polynomial_remainder_theorem_bundled_bound,
    polynomial_remainder_theorem_bundled_of_support_bounded_le,
    polynomial_support_bounded_by_monotone, polynomial_support_bound_exists
from polynomial.global_eval import polynomial_eval, polynomial_eval_eq_eval_bound_of_support_bounded

define polynomial_x_minus_a_polynomial_witness_at[R: CommRing](
    p: Polynomial[R],
    a: R,
    q: Polynomial[R],
    x: R
) -> Bool {
    polynomial_eval(p, x) = polynomial_eval(q, x) * (x - a)
}

/// A polynomial quotient witnesses global divisibility by the linear factor `X - a`.
define polynomial_x_minus_a_polynomial_witness[R: CommRing](
    p: Polynomial[R],
    a: R,
    q: Polynomial[R]
) -> Bool {
    forall(x: R) {
        polynomial_x_minus_a_polynomial_witness_at(p, a, q, x)
    }
}

/// Global polynomial-witness divisibility by the linear factor `X - a`.
define polynomial_divisible_by_x_minus_a_polynomial[R: CommRing](p: Polynomial[R], a: R) -> Bool {
    exists(q: Polynomial[R]) {
        polynomial_x_minus_a_polynomial_witness(p, a, q)
    }
}

/// Applying a global polynomial quotient witness at one evaluation point.
theorem polynomial_x_minus_a_polynomial_witness_apply[R: CommRing](
    p: Polynomial[R],
    a: R,
    q: Polynomial[R],
    x: R
) {
    polynomial_x_minus_a_polynomial_witness(p, a, q) implies
    polynomial_eval(p, x) = polynomial_eval(q, x) * (x - a)
} by {
    if polynomial_x_minus_a_polynomial_witness(p, a, q) {
        polynomial_x_minus_a_polynomial_witness(p, a, q) = forall(y: R) {
            polynomial_x_minus_a_polynomial_witness_at(p, a, q, y)
        }
        polynomial_x_minus_a_polynomial_witness_at(p, a, q, x)
        polynomial_x_minus_a_polynomial_witness_at(p, a, q, x) =
            (polynomial_eval(p, x) = polynomial_eval(q, x) * (x - a))
        polynomial_eval(p, x) = polynomial_eval(q, x) * (x - a)
    }
}

/// At a valid support bound, the bundled synthetic quotient is a global witness for a root.
theorem polynomial_quotient_global_witness_of_support_bounded[R: CommRing](p: Polynomial[R], a: R, n: Nat) {
    polynomial_support_bounded_by(p, n) and polynomial_eval(p, a) = R.0 implies
    polynomial_x_minus_a_polynomial_witness(p, a, polynomial_quotient(p, a, n))
} by {
    if polynomial_support_bounded_by(p, n) and polynomial_eval(p, a) = R.0 {
        let q = polynomial_quotient(p, a, n)
        polynomial_quotient_support_bounded_by(p, a, n)
        polynomial_support_bounded_by(q, n)
        polynomial_eval_eq_eval_bound_of_support_bounded(p, a, n)
        polynomial_eval(p, a) = polynomial_eval_bound(p, a, n)
        polynomial_eval_bound(p, a, n) = R.0
        forall(x: R) {
            polynomial_eval_eq_eval_bound_of_support_bounded(p, x, n)
            polynomial_eval_eq_eval_bound_of_support_bounded(q, x, n)
            polynomial_eval(p, x) = polynomial_eval_bound(p, x, n)
            polynomial_eval(q, x) = polynomial_eval_bound(q, x, n)
            polynomial_remainder_theorem_bundled_bound(p, a, x, n)
            polynomial_eval_bound(p, x, n) =
                polynomial_eval_bound(p, a, n) + (x - a) * polynomial_eval_bound(q, x, n)
            polynomial_eval_bound(p, a, n) = R.0
            R.0 + (x - a) * polynomial_eval_bound(q, x, n) =
                (x - a) * polynomial_eval_bound(q, x, n)
            (x - a) * polynomial_eval_bound(q, x, n) =
                polynomial_eval_bound(q, x, n) * (x - a)
            polynomial_eval_bound(q, x, n) = polynomial_eval(q, x)
            polynomial_eval_bound(q, x, n) * (x - a) = polynomial_eval(q, x) * (x - a)
            polynomial_eval(p, x) = polynomial_eval(q, x) * (x - a)
        }
        polynomial_x_minus_a_polynomial_witness(p, a, q) = forall(x: R) {
            polynomial_x_minus_a_polynomial_witness_at(p, a, q, x)
        }
        polynomial_x_minus_a_polynomial_witness(p, a, q)
        q = polynomial_quotient(p, a, n)
        polynomial_x_minus_a_polynomial_witness(p, a, polynomial_quotient(p, a, n))
    }
}

/// At any larger valid support bound, the bundled synthetic quotient is a global witness for a root.
theorem polynomial_quotient_global_witness_of_support_bounded_le[R: CommRing](
    p: Polynomial[R],
    a: R,
    n: Nat,
    m: Nat
) {
    polynomial_support_bounded_by(p, n) and n <= m and polynomial_eval(p, a) = R.0 implies
    polynomial_x_minus_a_polynomial_witness(p, a, polynomial_quotient(p, a, m))
} by {
    if polynomial_support_bounded_by(p, n) and n <= m and polynomial_eval(p, a) = R.0 {
        let q = polynomial_quotient(p, a, m)
        polynomial_support_bounded_by_monotone(p, n, m)
        polynomial_support_bounded_by(p, m)
        polynomial_quotient_support_bounded_by(p, a, m)
        polynomial_support_bounded_by(q, m)
        polynomial_eval_eq_eval_bound_of_support_bounded(p, a, n)
        polynomial_eval(p, a) = polynomial_eval_bound(p, a, n)
        polynomial_eval_bound(p, a, n) = R.0
        forall(x: R) {
            polynomial_eval_eq_eval_bound_of_support_bounded(p, x, m)
            polynomial_eval_eq_eval_bound_of_support_bounded(q, x, m)
            polynomial_eval(p, x) = polynomial_eval_bound(p, x, m)
            polynomial_eval(q, x) = polynomial_eval_bound(q, x, m)
            polynomial_remainder_theorem_bundled_of_support_bounded_le(p, a, x, n, m)
            polynomial_eval_bound(p, x, m) =
                polynomial_eval_bound(p, a, n) + (x - a) * polynomial_eval_bound(q, x, m)
            polynomial_eval_bound(p, a, n) = R.0
            R.0 + (x - a) * polynomial_eval_bound(q, x, m) =
                (x - a) * polynomial_eval_bound(q, x, m)
            (x - a) * polynomial_eval_bound(q, x, m) =
                polynomial_eval_bound(q, x, m) * (x - a)
            polynomial_eval_bound(q, x, m) = polynomial_eval(q, x)
            polynomial_eval_bound(q, x, m) * (x - a) = polynomial_eval(q, x) * (x - a)
            polynomial_eval(p, x) = polynomial_eval(q, x) * (x - a)
        }
        polynomial_x_minus_a_polynomial_witness(p, a, q) = forall(x: R) {
            polynomial_x_minus_a_polynomial_witness_at(p, a, q, x)
        }
        polynomial_x_minus_a_polynomial_witness(p, a, q)
        q = polynomial_quotient(p, a, m)
        polynomial_x_minus_a_polynomial_witness(p, a, polynomial_quotient(p, a, m))
    }
}

/// A global root gives global divisibility by `X - a` with a polynomial witness.
theorem polynomial_factor_theorem_forward_global[R: CommRing](p: Polynomial[R], a: R) {
    polynomial_eval(p, a) = R.0 implies polynomial_divisible_by_x_minus_a_polynomial(p, a)
} by {
    if polynomial_eval(p, a) = R.0 {
        polynomial_support_bound_exists(p)
        let n: Nat satisfy {
            polynomial_support_bounded_by(p, n)
        }
        polynomial_quotient_global_witness_of_support_bounded(p, a, n)
        polynomial_x_minus_a_polynomial_witness(p, a, polynomial_quotient(p, a, n))
        exists(q: Polynomial[R]) {
            q = polynomial_quotient(p, a, n) and polynomial_x_minus_a_polynomial_witness(p, a, q)
        }
        polynomial_divisible_by_x_minus_a_polynomial(p, a)
    }
}

/// Global divisibility by `X - a` forces `a` to be a global root.
theorem polynomial_factor_theorem_reverse_global[R: CommRing](p: Polynomial[R], a: R) {
    polynomial_divisible_by_x_minus_a_polynomial(p, a) implies polynomial_eval(p, a) = R.0
} by {
    if polynomial_divisible_by_x_minus_a_polynomial(p, a) {
        let q: Polynomial[R] satisfy {
            polynomial_x_minus_a_polynomial_witness(p, a, q)
        }
        polynomial_x_minus_a_polynomial_witness_apply(p, a, q, a)
        polynomial_eval(p, a) = polynomial_eval(q, a) * (a - a)
        a - a = a + -a
        a + -a = R.0
        a - a = R.0
        polynomial_eval(q, a) * R.0 = R.0
        polynomial_eval(p, a) = R.0
    }
}

/// The global polynomial factor theorem for the polynomial-witness divisibility predicate.
theorem polynomial_factor_theorem_global[R: CommRing](p: Polynomial[R], a: R) {
    polynomial_eval(p, a) = R.0 = polynomial_divisible_by_x_minus_a_polynomial(p, a)
} by {
    if polynomial_eval(p, a) = R.0 {
        polynomial_factor_theorem_forward_global(p, a)
        polynomial_divisible_by_x_minus_a_polynomial(p, a)
    }
    if polynomial_divisible_by_x_minus_a_polynomial(p, a) {
        polynomial_factor_theorem_reverse_global(p, a)
        polynomial_eval(p, a) = R.0
    }
}
