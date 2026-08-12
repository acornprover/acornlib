/// Roots of the square of a linear factor: the square `(X - a)^2` has the
/// single root `a`, and a double root (divisibility by the square) forces a
/// root.

from nat import Nat
from comm_ring import CommRing
from algebra.field.field import Field, field_mul_eq_zero
from polynomial import Polynomial, polynomial_eval, polynomial_eval_mul, polynomial_mul
from polynomial.polynomial_vieta import linear_polynomial, linear_polynomial_eval,
    linear_polynomial_neg_root
from polynomial.polynomial_roots import polynomial_sub_eq_zero_iff, polynomial_root_linear_one,
    polynomial_root_product_iff
from polynomial.polynomial_divisibility import polynomial_divides, polynomial_divides_eval_witness

numerals Nat

/// A polynomial has a double root at `a` when the square of `X - a` divides
/// it.
define polynomial_double_root[R: CommRing](p: Polynomial[R], a: R) -> Bool {
    polynomial_divides(polynomial_mul(linear_polynomial(R.1, -a), linear_polynomial(R.1, -a)),
        p)
}

/// The square of the linear factor `X - a` vanishes only at `a`.
theorem polynomial_square_root_unique[F: Field](a: F, x: F) {
    (polynomial_eval(polynomial_mul(linear_polynomial(F.1, -a),
        linear_polynomial(F.1, -a)), x) = F.0) = (x = a)
} by {
    polynomial_root_product_iff(linear_polynomial(F.1, -a), linear_polynomial(F.1, -a), x)
    (polynomial_eval(polynomial_mul(linear_polynomial(F.1, -a),
        linear_polynomial(F.1, -a)), x) = F.0) =
        (polynomial_eval(linear_polynomial(F.1, -a), x) = F.0 or
            polynomial_eval(linear_polynomial(F.1, -a), x) = F.0)
    polynomial_root_linear_one(a, x)
    (polynomial_eval(linear_polynomial(F.1, -a), x) = F.0) = (x = a)
    (polynomial_eval(linear_polynomial(F.1, -a), x) = F.0 or
        polynomial_eval(linear_polynomial(F.1, -a), x) = F.0) = (x = a)
    (polynomial_eval(polynomial_mul(linear_polynomial(F.1, -a),
        linear_polynomial(F.1, -a)), x) = F.0) = (x = a)
}

/// The square of `X - a` vanishes at `a`.
theorem polynomial_square_root_at_a[R: CommRing](a: R) {
    polynomial_eval(polynomial_mul(linear_polynomial(R.1, -a),
        linear_polynomial(R.1, -a)), a) = R.0
} by {
    linear_polynomial_neg_root(a)
    polynomial_eval(linear_polynomial(R.1, -a), a) = R.0
    polynomial_eval_mul(linear_polynomial(R.1, -a), linear_polynomial(R.1, -a), a)
    polynomial_eval(polynomial_mul(linear_polynomial(R.1, -a),
        linear_polynomial(R.1, -a)), a) =
        polynomial_eval(linear_polynomial(R.1, -a), a) *
        polynomial_eval(linear_polynomial(R.1, -a), a)
    polynomial_eval(linear_polynomial(R.1, -a), a) *
        polynomial_eval(linear_polynomial(R.1, -a), a) = R.0 * R.0
    R.0 * R.0 = R.0
    polynomial_eval(polynomial_mul(linear_polynomial(R.1, -a),
        linear_polynomial(R.1, -a)), a) = R.0
}

/// A double root at `a` forces `a` to be a root.
theorem polynomial_double_root_implies_root[R: CommRing](p: Polynomial[R], a: R) {
    polynomial_double_root(p, a) implies polynomial_eval(p, a) = R.0
} by {
    if polynomial_double_root(p, a) {
        let q: Polynomial[R] satisfy {
            polynomial_mul(polynomial_mul(linear_polynomial(R.1, -a),
                linear_polynomial(R.1, -a)), q) = p
        }
        polynomial_divides_eval_witness(
            polynomial_mul(linear_polynomial(R.1, -a), linear_polynomial(R.1, -a)), p, q, a)
        polynomial_eval(p, a) =
            polynomial_eval(polynomial_mul(linear_polynomial(R.1, -a),
                linear_polynomial(R.1, -a)), a) * polynomial_eval(q, a)
        polynomial_square_root_at_a(a)
        polynomial_eval(polynomial_mul(linear_polynomial(R.1, -a),
            linear_polynomial(R.1, -a)), a) = R.0
        polynomial_eval(p, a) = R.0 * polynomial_eval(q, a)
        R.0 * polynomial_eval(q, a) = R.0
        polynomial_eval(p, a) = R.0
    }
}

/// A double root of `p` at `a` factorizes the value of `p` by the square at
/// every point.
theorem polynomial_double_root_eval[R: CommRing](p: Polynomial[R], a: R, x: R) {
    polynomial_double_root(p, a) implies
    exists(q: Polynomial[R]) {
        polynomial_eval(p, x) =
            polynomial_eval(polynomial_mul(linear_polynomial(R.1, -a),
                linear_polynomial(R.1, -a)), x) * polynomial_eval(q, x)
    }
} by {
    if polynomial_double_root(p, a) {
        let q: Polynomial[R] satisfy {
            polynomial_mul(polynomial_mul(linear_polynomial(R.1, -a),
                linear_polynomial(R.1, -a)), q) = p
        }
        polynomial_divides_eval_witness(
            polynomial_mul(linear_polynomial(R.1, -a), linear_polynomial(R.1, -a)), p, q, x)
        polynomial_eval(p, x) =
            polynomial_eval(polynomial_mul(linear_polynomial(R.1, -a),
                linear_polynomial(R.1, -a)), x) * polynomial_eval(q, x)
        exists(s: Polynomial[R]) {
            s = q and polynomial_eval(p, x) =
                polynomial_eval(polynomial_mul(linear_polynomial(R.1, -a),
                    linear_polynomial(R.1, -a)), x) * polynomial_eval(s, x)
        }
        exists(s: Polynomial[R]) {
            polynomial_eval(p, x) =
                polynomial_eval(polynomial_mul(linear_polynomial(R.1, -a),
                    linear_polynomial(R.1, -a)), x) * polynomial_eval(s, x)
        }
    }
}
