/// The monomial decomposition of a polynomial: every polynomial is the sum
/// of its monomials, and the initial segment up to a support bound recovers
/// the polynomial.

from nat import Nat, alt_induction, lt_suc, lt_imp_lt_suc, lt_or_lte, not_lt_zero,
    lt_not_ref, lte_antisymm
from order import lt_imp_lte
from semiring import Semiring
from polynomial import Polynomial, polynomial_monomial, polynomial_monomial_coeff_self,
    polynomial_monomial_coeff_of_ne, polynomial_zero_coeff, polynomial_ext_pointwise,
    polynomial_add_coeff, polynomial_support_bounded_by, polynomial_support_bound_exists,
    polynomial_support_bounded_by_apply
from polynomial.polynomial_eval_deep import polynomial_monomial_support_bounded_by_suc

numerals Nat

/// The attribute monomial is the same polynomial as the named monomial.
lemma structure_monomial_alias[R: Semiring](n: Nat, r: R) {
    Polynomial[R].monomial(n, r) = polynomial_monomial(n, r)
} by {
    Polynomial[R].monomial(n, r) = polynomial_monomial(n, r)
}

/// A monomial has its coefficient at its exponent.
lemma structure_monomial_coeff_self[R: Semiring](n: Nat, r: R) {
    polynomial_monomial(n, r).coeff(n) = r
} by {
    structure_monomial_alias(n, r)
    Polynomial[R].monomial(n, r) = polynomial_monomial(n, r)
    polynomial_monomial_coeff_self(n, r)
    Polynomial[R].monomial(n, r).coeff(n) = r
    polynomial_monomial(n, r).coeff(n) = r
}

/// A monomial vanishes away from its exponent.
lemma structure_monomial_coeff_of_ne[R: Semiring](n: Nat, r: R, k: Nat) {
    k != n implies polynomial_monomial(n, r).coeff(k) = R.0
} by {
    if k != n {
        structure_monomial_alias(n, r)
        Polynomial[R].monomial(n, r) = polynomial_monomial(n, r)
        polynomial_monomial_coeff_of_ne[R](n, r, k)
        Polynomial[R].monomial(n, r).coeff(k) = R.0
        polynomial_monomial(n, r).coeff(k) = R.0
    }
}

/// The sum of the monomials `coeff(0) X^0` through `coeff(n - 1) X^(n - 1)`.
define polynomial_monomial_partial_sum[R: Semiring](p: Polynomial[R], n: Nat) -> Polynomial[R] {
    match n {
        Nat.zero {
            Polynomial[R].zero
        }
        Nat.suc(pred) {
            polynomial_monomial_partial_sum(p, pred) + polynomial_monomial(pred, p.coeff(pred))
        }
    }
}

/// A coefficient of the partial monomial sum at or beyond the bound is zero.
lemma monomial_partial_sum_coeff_ge[R: Semiring](p: Polynomial[R], n: Nat, j: Nat) {
    not j < n implies polynomial_monomial_partial_sum(p, n).coeff(j) = R.0
} by {
    define statement(k: Nat) -> Bool {
        forall(i: Nat) {
            not i < k implies polynomial_monomial_partial_sum(p, k).coeff(i) = R.0
        }
    }

    forall(i: Nat) {
        if not i < Nat.0 {
            polynomial_zero_coeff[R](i)
            polynomial_monomial_partial_sum(p, Nat.0).coeff(i) =
                Polynomial[R].zero.coeff(i)
            polynomial_monomial_partial_sum(p, Nat.0).coeff(i) = R.0
        }
    }
    statement(Nat.0)

    forall(k: Nat) {
        if statement(k) {
            forall(i: Nat) {
                if not i < k.suc {
                    polynomial_add_coeff(polynomial_monomial_partial_sum(p, k),
                        polynomial_monomial(k, p.coeff(k)), i)
                    polynomial_monomial_partial_sum(p, k.suc).coeff(i) =
                        polynomial_monomial_partial_sum(p, k).coeff(i) +
                        polynomial_monomial(k, p.coeff(k)).coeff(i)
                    if i < k {
                        lt_imp_lt_suc(i, k)
                        i < k.suc
                        false
                    }
                    not i < k
                    statement(k) = forall(h: Nat) {
                        not h < k implies polynomial_monomial_partial_sum(p, k).coeff(h) = R.0
                    }
                    polynomial_monomial_partial_sum(p, k).coeff(i) = R.0
                    i != k
                    if i = k {
                        lt_suc(k)
                        k < k.suc
                        false
                    }
                    structure_monomial_coeff_of_ne(k, p.coeff(k), i)
                    polynomial_monomial(k, p.coeff(k)).coeff(i) = R.0
                    polynomial_monomial_partial_sum(p, k.suc).coeff(i) = R.0 + R.0
                    R.0 + R.0 = R.0
                    polynomial_monomial_partial_sum(p, k.suc).coeff(i) = R.0
                }
            }
            statement(k.suc)
        }
    }

    statement(Nat.0) and forall(k: Nat) { statement(k) implies statement(k.suc) }
    alt_induction(statement)
    forall(k: Nat) { statement(k) }
    statement(n)
    statement(n) = forall(i: Nat) {
        not i < n implies polynomial_monomial_partial_sum(p, n).coeff(i) = R.0
    }
    polynomial_monomial_partial_sum(p, n).coeff(j) = R.0
}

/// A coefficient of the partial monomial sum below the bound is the
/// coefficient of the polynomial.
lemma monomial_partial_sum_coeff_lt[R: Semiring](p: Polynomial[R], n: Nat, j: Nat) {
    j < n implies polynomial_monomial_partial_sum(p, n).coeff(j) = p.coeff(j)
} by {
    define statement(k: Nat) -> Bool {
        forall(i: Nat) {
            i < k implies polynomial_monomial_partial_sum(p, k).coeff(i) = p.coeff(i)
        }
    }

    forall(i: Nat) {
        if i < Nat.0 {
            not_lt_zero(i)
            false
        }
    }
    statement(Nat.0)

    forall(k: Nat) {
        if statement(k) {
            forall(i: Nat) {
                if i < k.suc {
                    polynomial_add_coeff(polynomial_monomial_partial_sum(p, k),
                        polynomial_monomial(k, p.coeff(k)), i)
                    polynomial_monomial_partial_sum(p, k.suc).coeff(i) =
                        polynomial_monomial_partial_sum(p, k).coeff(i) +
                        polynomial_monomial(k, p.coeff(k)).coeff(i)
                    if i < k {
                        statement(k) = forall(h: Nat) {
                            h < k implies polynomial_monomial_partial_sum(p, k).coeff(h) = p.coeff(h)
                        }
                        polynomial_monomial_partial_sum(p, k).coeff(i) = p.coeff(i)
                        i != k
                        structure_monomial_coeff_of_ne(k, p.coeff(k), i)
                        polynomial_monomial(k, p.coeff(k)).coeff(i) = R.0
                        polynomial_monomial_partial_sum(p, k.suc).coeff(i) = p.coeff(i) + R.0
                        p.coeff(i) + R.0 = p.coeff(i)
                        polynomial_monomial_partial_sum(p, k.suc).coeff(i) = p.coeff(i)
                    } else {
                        not i < k
                        lt_or_lte(i, k)
                        if i < k {
                            false
                        }
                        k <= i
                        lt_imp_lte(i, k)
                        i <= k
                        lte_antisymm(i, k)
                        i = k
                        lt_not_ref(k)
                        not k < k
                        statement(k) = forall(h: Nat) {
                            not h < k implies polynomial_monomial_partial_sum(p, k).coeff(h) = R.0
                        }
                        polynomial_monomial_partial_sum(p, k).coeff(k) = R.0
                        i = k
                        polynomial_monomial_partial_sum(p, k).coeff(i) = R.0
                        structure_monomial_coeff_self(k, p.coeff(k))
                        polynomial_monomial(k, p.coeff(k)).coeff(k) = p.coeff(k)
                        polynomial_monomial_partial_sum(p, k.suc).coeff(i) = R.0 + p.coeff(i)
                        R.0 + p.coeff(i) = p.coeff(i)
                        polynomial_monomial_partial_sum(p, k.suc).coeff(i) = p.coeff(i)
                    }
                }
            }
            statement(k.suc)
        }
    }

    statement(Nat.0) and forall(k: Nat) { statement(k) implies statement(k.suc) }
    alt_induction(statement)
    forall(k: Nat) { statement(k) }
    statement(n)
    statement(n) = forall(i: Nat) {
        i < n implies polynomial_monomial_partial_sum(p, n).coeff(i) = p.coeff(i)
    }
    polynomial_monomial_partial_sum(p, n).coeff(j) = p.coeff(j)
}

/// A polynomial supported below `n` is the sum of its first `n` monomials.
theorem polynomial_eq_monomial_partial_sum_of_support_bounded[R: Semiring](
    p: Polynomial[R], n: Nat
) {
    polynomial_support_bounded_by(p, n) implies p = polynomial_monomial_partial_sum(p, n)
} by {
    if polynomial_support_bounded_by(p, n) {
        forall(j: Nat) {
            if j < n {
                monomial_partial_sum_coeff_lt(p, n, j)
                polynomial_monomial_partial_sum(p, n).coeff(j) = p.coeff(j)
                p.coeff(j) = polynomial_monomial_partial_sum(p, n).coeff(j)
            } else {
                not j < n
                monomial_partial_sum_coeff_ge(p, n, j)
                polynomial_monomial_partial_sum(p, n).coeff(j) = R.0
                polynomial_support_bounded_by_apply(p, n, j)
                p.coeff(j) = R.0
                p.coeff(j) = polynomial_monomial_partial_sum(p, n).coeff(j)
            }
        }
        polynomial_ext_pointwise(p, polynomial_monomial_partial_sum(p, n))
        p = polynomial_monomial_partial_sum(p, n)
    }
}

/// Every polynomial is the sum of its monomials up to a support bound.
theorem polynomial_eq_monomial_partial_sum[R: Semiring](p: Polynomial[R]) {
    exists(n: Nat) {
        p = polynomial_monomial_partial_sum(p, n)
    }
} by {
    polynomial_support_bound_exists(p)
    let n: Nat satisfy {
        polynomial_support_bounded_by(p, n)
    }
    polynomial_eq_monomial_partial_sum_of_support_bounded(p, n)
    p = polynomial_monomial_partial_sum(p, n)
    exists(m: Nat) {
        m = n and p = polynomial_monomial_partial_sum(p, m)
    }
}
