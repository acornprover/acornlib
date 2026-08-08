/// Hom-style API for evaluation of a polynomial at a fixed point.
///
/// The polynomial package currently has named operations for multiplication and
/// additive inverse, but not a bundled semiring/ring instance on `Polynomial[R]`.
/// This module packages the preservation facts for the evaluation function in
/// the shape needed by homomorphism constructors once such an instance is
/// available.

from semiring import Semiring
from algebra.ring.ring import Ring
from comm_ring import CommRing
from polynomial.base import Polynomial
from polynomial.mul import polynomial_mul
from polynomial.global_eval import polynomial_eval, polynomial_eval_zero, polynomial_eval_constant,
    polynomial_eval_one, polynomial_eval_add, polynomial_eval_neg, polynomial_eval_sub
from polynomial.eval_mul import polynomial_eval_mul

/// Evaluation at a fixed point, viewed as a function from polynomials to the coefficient ring.
define polynomial_eval_at[R: Semiring](x: R, p: Polynomial[R]) -> R {
    polynomial_eval(p, x)
}

/// Evaluation at a fixed point sends the zero polynomial to zero.
theorem polynomial_eval_at_zero[R: Semiring](x: R) {
    polynomial_eval_at(x, Polynomial[R].zero) = R.0
} by {
    polynomial_eval_zero(x)
}

/// Evaluation at a fixed point sends a constant polynomial to the represented coefficient.
theorem polynomial_eval_at_constant[R: Semiring](x: R, r: R) {
    polynomial_eval_at(x, Polynomial[R].constant(r)) = r
} by {
    polynomial_eval_constant(r, x)
}

/// Evaluation at a fixed point sends the one polynomial to one.
theorem polynomial_eval_at_one[R: Semiring](x: R) {
    polynomial_eval_at(x, Polynomial[R].one) = R.1
} by {
    polynomial_eval_one(x)
}

/// Evaluation at a fixed point preserves polynomial addition.
theorem polynomial_eval_at_add[R: Semiring](x: R, p: Polynomial[R], q: Polynomial[R]) {
    polynomial_eval_at(x, p + q) = polynomial_eval_at(x, p) + polynomial_eval_at(x, q)
} by {
    polynomial_eval_add(p, q, x)
}

/// Evaluation at a fixed point preserves the polynomial product.
theorem polynomial_eval_at_mul[R: CommRing](x: R, p: Polynomial[R], q: Polynomial[R]) {
    polynomial_eval_at(x, polynomial_mul(p, q)) = polynomial_eval_at(x, p) * polynomial_eval_at(x, q)
} by {
    polynomial_eval_mul(p, q, x)
}

/// Evaluation at a fixed point preserves coefficientwise additive inverse.
theorem polynomial_eval_at_neg[R: Ring](x: R, p: Polynomial[R]) {
    polynomial_eval_at(x, p.neg) = -polynomial_eval_at(x, p)
} by {
    polynomial_eval_neg(p, x)
}

/// Evaluation at a fixed point preserves coefficientwise subtraction.
theorem polynomial_eval_at_sub[R: Ring](x: R, p: Polynomial[R], q: Polynomial[R]) {
    polynomial_eval_at(x, p.sub(q)) = polynomial_eval_at(x, p) - polynomial_eval_at(x, q)
} by {
    polynomial_eval_sub(p, q, x)
}
