/// Deepening polynomial theory: evaluation laws, roots and the factor theorem,
/// degree bounds, and the zero polynomial, built on the bounded and global
/// evaluation infrastructure.

from nat import Nat, alt_induction, alt_suc_ne_zero, lt_suc, lt_imp_lt_suc, lte_suc_suc, sub_pos,
    sub_self
from semiring import Semiring
from comm_ring import CommRing
from algebra.comm_semigroup import CommSemigroup
from algebra.semigroup import Semigroup
from algebra.field.field import Field, field_mul_eq_zero
from list import partial, partial_one, partial_split_last, partial_zero
from polynomial.base import Polynomial, polynomial_zero_coeff, polynomial_eq_zero_of_coeff_zero,
    polynomial_coeff_zero_of_eq_zero, polynomial_ext_pointwise, polynomial_one_coeff_zero,
    polynomial_one_coeff_of_ne_zero
from polynomial.mul import polynomial_mul, polynomial_mul_coeff, polynomial_mul_coeff_apply,
    polynomial_mul_term_coeff, polynomial_mul_support_bounded_by_add
from polynomial.eval import polynomial_support_bounded_by, polynomial_support_bounded_by_monotone,
    polynomial_quotient, polynomial_zero_support_bounded_by_zero,
    polynomial_constant_support_bounded_by_one
from polynomial.global_eval import polynomial_eval, polynomial_eval_zero, polynomial_eval_constant,
    polynomial_eval_add
from polynomial.eval_mul import polynomial_eval_mul
from polynomial.global_factor import polynomial_x_minus_a_polynomial_witness,
    polynomial_x_minus_a_polynomial_witness_apply, polynomial_divisible_by_x_minus_a_polynomial,
    polynomial_quotient_global_witness_of_support_bounded
from polynomial.polynomial_ring import polynomial_degree_le, polynomial_monomial_support_bounded_by_suc,
    linear_polynomial, linear_polynomial_eval, polynomial_remainder_theorem_global

numerals Nat

// ---------------------------------------------------------------------------
// Evaluation properties
// ---------------------------------------------------------------------------

/// Equal polynomials evaluate equally at every point.
theorem polynomial_eval_eq_of_eq[R: Semiring](p: Polynomial[R], q: Polynomial[R], x: R) {
    p = q implies polynomial_eval(p, x) = polynomial_eval(q, x)
} by {
    if p = q {
        polynomial_eval(p, x) = polynomial_eval(q, x)
    }
}

/// Polynomial equality implies equality of all evaluations: evaluation is
/// extensional.
theorem polynomial_eval_ext[R: Semiring](p: Polynomial[R], q: Polynomial[R]) {
    p = q implies forall(x: R) { polynomial_eval(p, x) = polynomial_eval(q, x) }
} by {
    if p = q {
        forall(x: R) {
            polynomial_eval(p, x) = polynomial_eval(q, x)
        }
    }
}

/// Pointwise coefficient equality implies equality of evaluations.
///
/// The converse — equal evaluations at every point force coefficient
/// equality — is false over general rings (e.g. `X^2 - X` vanishes on every
/// element of the two-element field), so only this direction is stated.
theorem polynomial_eval_eq_of_coeff_eq[R: Semiring](p: Polynomial[R], q: Polynomial[R], x: R) {
    (forall(n: Nat) { p.coeff(n) = q.coeff(n) }) implies polynomial_eval(p, x) = polynomial_eval(q, x)
} by {
    if forall(n: Nat) { p.coeff(n) = q.coeff(n) } {
        polynomial_ext_pointwise(p, q)
        p = q
        polynomial_eval_eq_of_eq(p, q, x)
        polynomial_eval(p, x) = polynomial_eval(q, x)
    }
}

/// Evaluation of a constant plus a polynomial is the constant plus the value.
theorem polynomial_eval_constant_add[R: Semiring](r: R, p: Polynomial[R], x: R) {
    polynomial_eval(Polynomial[R].constant(r) + p, x) = r + polynomial_eval(p, x)
} by {
    polynomial_eval_add(Polynomial[R].constant(r), p, x)
    polynomial_eval(Polynomial[R].constant(r) + p, x) =
        polynomial_eval(Polynomial[R].constant(r), x) + polynomial_eval(p, x)
    polynomial_eval_constant(r, x)
    polynomial_eval(Polynomial[R].constant(r), x) = r
    polynomial_eval(Polynomial[R].constant(r) + p, x) = r + polynomial_eval(p, x)
}

/// Evaluation of a scalar multiple: a constant polynomial times a polynomial
/// evaluates to the scalar times the value.
theorem polynomial_eval_constant_mul[R: CommRing](r: R, p: Polynomial[R], x: R) {
    polynomial_eval(polynomial_mul(Polynomial[R].constant(r), p), x) = r * polynomial_eval(p, x)
} by {
    polynomial_eval_mul(Polynomial[R].constant(r), p, x)
    polynomial_eval(polynomial_mul(Polynomial[R].constant(r), p), x) =
        polynomial_eval(Polynomial[R].constant(r), x) * polynomial_eval(p, x)
    polynomial_eval_constant(r, x)
    polynomial_eval(Polynomial[R].constant(r), x) = r
    polynomial_eval(polynomial_mul(Polynomial[R].constant(r), p), x) = r * polynomial_eval(p, x)
}

/// The evaluation of an iterated product is independent of the bracketing:
/// associativity of multiplication at the evaluation level.
theorem polynomial_eval_mul_assoc[R: CommRing](
    p: Polynomial[R],
    q: Polynomial[R],
    r: Polynomial[R],
    x: R
) {
    polynomial_eval(polynomial_mul(polynomial_mul(p, q), r), x) =
        polynomial_eval(polynomial_mul(p, polynomial_mul(q, r)), x)
} by {
    polynomial_eval_mul(polynomial_mul(p, q), r, x)
    polynomial_eval(polynomial_mul(polynomial_mul(p, q), r), x) =
        polynomial_eval(polynomial_mul(p, q), x) * polynomial_eval(r, x)
    polynomial_eval_mul(p, q, x)
    polynomial_eval(polynomial_mul(p, q), x) = polynomial_eval(p, x) * polynomial_eval(q, x)
    polynomial_eval(polynomial_mul(polynomial_mul(p, q), r), x) =
        (polynomial_eval(p, x) * polynomial_eval(q, x)) * polynomial_eval(r, x)
    Semigroup.mul_associative[R](polynomial_eval(p, x), polynomial_eval(q, x), polynomial_eval(r, x))
    (polynomial_eval(p, x) * polynomial_eval(q, x)) * polynomial_eval(r, x) =
        polynomial_eval(p, x) * (polynomial_eval(q, x) * polynomial_eval(r, x))
    polynomial_eval_mul(q, r, x)
    polynomial_eval(polynomial_mul(q, r), x) = polynomial_eval(q, x) * polynomial_eval(r, x)
    polynomial_eval_mul(p, polynomial_mul(q, r), x)
    polynomial_eval(polynomial_mul(p, polynomial_mul(q, r)), x) =
        polynomial_eval(p, x) * polynomial_eval(polynomial_mul(q, r), x)
    polynomial_eval(polynomial_mul(p, polynomial_mul(q, r)), x) =
        polynomial_eval(p, x) * (polynomial_eval(q, x) * polynomial_eval(r, x))
    polynomial_eval(polynomial_mul(polynomial_mul(p, q), r), x) =
        polynomial_eval(polynomial_mul(p, polynomial_mul(q, r)), x)
}

/// The evaluation of a product is independent of the factor order:
/// commutativity of multiplication at the evaluation level.
theorem polynomial_eval_mul_comm[R: CommRing](p: Polynomial[R], q: Polynomial[R], x: R) {
    polynomial_eval(polynomial_mul(p, q), x) = polynomial_eval(polynomial_mul(q, p), x)
} by {
    polynomial_eval_mul(p, q, x)
    polynomial_eval(polynomial_mul(p, q), x) = polynomial_eval(p, x) * polynomial_eval(q, x)
    polynomial_eval_mul(q, p, x)
    polynomial_eval(polynomial_mul(q, p), x) = polynomial_eval(q, x) * polynomial_eval(p, x)
    CommSemigroup.commutative[R](polynomial_eval(p, x), polynomial_eval(q, x))
    polynomial_eval(p, x) * polynomial_eval(q, x) = polynomial_eval(q, x) * polynomial_eval(p, x)
    polynomial_eval(polynomial_mul(p, q), x) = polynomial_eval(polynomial_mul(q, p), x)
}

/// The value of a sum of products is the sum of the values of the products:
/// the distributive law at the evaluation level.
theorem polynomial_eval_mul_add_distrib[R: CommRing](
    p: Polynomial[R],
    q: Polynomial[R],
    r: Polynomial[R],
    s: Polynomial[R],
    x: R
) {
    polynomial_eval(polynomial_mul(p, q) + polynomial_mul(r, s), x) =
        polynomial_eval(p, x) * polynomial_eval(q, x) + polynomial_eval(r, x) * polynomial_eval(s, x)
} by {
    polynomial_eval_add(polynomial_mul(p, q), polynomial_mul(r, s), x)
    polynomial_eval(polynomial_mul(p, q) + polynomial_mul(r, s), x) =
        polynomial_eval(polynomial_mul(p, q), x) + polynomial_eval(polynomial_mul(r, s), x)
    polynomial_eval_mul(p, q, x)
    polynomial_eval(polynomial_mul(p, q), x) = polynomial_eval(p, x) * polynomial_eval(q, x)
    polynomial_eval_mul(r, s, x)
    polynomial_eval(polynomial_mul(r, s), x) = polynomial_eval(r, x) * polynomial_eval(s, x)
    polynomial_eval(polynomial_mul(p, q) + polynomial_mul(r, s), x) =
        polynomial_eval(p, x) * polynomial_eval(q, x) + polynomial_eval(r, x) * polynomial_eval(s, x)
}

// ---------------------------------------------------------------------------
// Roots and the factor theorem
// ---------------------------------------------------------------------------

/// At a valid support bound, a root of `p` exhibits the synthetic quotient as
/// the polynomial witness for divisibility by `X - a`.
theorem polynomial_factor_theorem_quotient_form[R: CommRing](p: Polynomial[R], a: R, n: Nat) {
    polynomial_support_bounded_by(p, n) and polynomial_eval(p, a) = R.0 implies
    polynomial_x_minus_a_polynomial_witness(p, a, polynomial_quotient(p, a, n))
} by {
    if polynomial_support_bounded_by(p, n) and polynomial_eval(p, a) = R.0 {
        polynomial_quotient_global_witness_of_support_bounded(p, a, n)
        polynomial_x_minus_a_polynomial_witness(p, a, polynomial_quotient(p, a, n))
    }
}

/// The factor theorem in value form: at a valid support bound, a root of `p`
/// factors `p(x)` as the quotient value times `x - a` at every point.
theorem polynomial_eval_of_root_factor[R: CommRing](p: Polynomial[R], a: R, n: Nat, x: R) {
    polynomial_support_bounded_by(p, n) and polynomial_eval(p, a) = R.0 implies
    polynomial_eval(p, x) = polynomial_eval(polynomial_quotient(p, a, n), x) * (x - a)
} by {
    if polynomial_support_bounded_by(p, n) and polynomial_eval(p, a) = R.0 {
        polynomial_factor_theorem_quotient_form(p, a, n)
        polynomial_x_minus_a_polynomial_witness(p, a, polynomial_quotient(p, a, n))
        polynomial_x_minus_a_polynomial_witness_apply(p, a, polynomial_quotient(p, a, n), x)
        polynomial_eval(p, x) = polynomial_eval(polynomial_quotient(p, a, n), x) * (x - a)
    }
}

/// The remainder theorem: `a` is a root of `p` exactly when `X - a` divides
/// `p`.
theorem polynomial_remainder_theorem_iff[R: CommRing](p: Polynomial[R], a: R) {
    polynomial_eval(p, a) = R.0 iff polynomial_divisible_by_x_minus_a_polynomial(p, a)
} by {
    polynomial_remainder_theorem_global(p, a)
}

/// A root of the first factor is a root of the product.
theorem polynomial_root_mul_left[R: CommRing](p: Polynomial[R], q: Polynomial[R], a: R) {
    polynomial_eval(p, a) = R.0 implies polynomial_eval(polynomial_mul(p, q), a) = R.0
} by {
    if polynomial_eval(p, a) = R.0 {
        polynomial_eval_mul(p, q, a)
        polynomial_eval(polynomial_mul(p, q), a) = polynomial_eval(p, a) * polynomial_eval(q, a)
        polynomial_eval(p, a) = R.0
        polynomial_eval(polynomial_mul(p, q), a) = R.0 * polynomial_eval(q, a)
        R.0 * polynomial_eval(q, a) = R.0
        polynomial_eval(polynomial_mul(p, q), a) = R.0
    }
}

/// A root of the second factor is a root of the product.
theorem polynomial_root_mul_right[R: CommRing](p: Polynomial[R], q: Polynomial[R], a: R) {
    polynomial_eval(q, a) = R.0 implies polynomial_eval(polynomial_mul(p, q), a) = R.0
} by {
    if polynomial_eval(q, a) = R.0 {
        polynomial_eval_mul(p, q, a)
        polynomial_eval(polynomial_mul(p, q), a) = polynomial_eval(p, a) * polynomial_eval(q, a)
        polynomial_eval(q, a) = R.0
        polynomial_eval(polynomial_mul(p, q), a) = polynomial_eval(p, a) * R.0
        polynomial_eval(p, a) * R.0 = R.0
        polynomial_eval(polynomial_mul(p, q), a) = R.0
    }
}

/// Over a field, a root of a product is a root of one of the factors: the
/// zero-product property for values.
theorem polynomial_root_of_product[F: Field](p: Polynomial[F], q: Polynomial[F], x: F) {
    polynomial_eval(polynomial_mul(p, q), x) = F.0 implies
    polynomial_eval(p, x) = F.0 or polynomial_eval(q, x) = F.0
} by {
    if polynomial_eval(polynomial_mul(p, q), x) = F.0 {
        polynomial_eval_mul(p, q, x)
        polynomial_eval(polynomial_mul(p, q), x) = polynomial_eval(p, x) * polynomial_eval(q, x)
        polynomial_eval(p, x) * polynomial_eval(q, x) = F.0
        field_mul_eq_zero(polynomial_eval(p, x), polynomial_eval(q, x))
        polynomial_eval(p, x) = F.0 or polynomial_eval(q, x) = F.0
    }
}

// ---------------------------------------------------------------------------
// Roots and translation (shifted evaluation)
// ---------------------------------------------------------------------------

/// The value of `p` after translating the evaluation point by `a`:
/// `q(x) = p(x + a)` in value form.
define polynomial_shift_eval[R: CommRing](p: Polynomial[R], a: R, x: R) -> R {
    polynomial_eval(p, x + a)
}

/// Translating by zero changes no evaluation.
theorem polynomial_shift_eval_zero[R: CommRing](p: Polynomial[R], x: R) {
    polynomial_shift_eval(p, R.0, x) = polynomial_eval(p, x)
} by {
    polynomial_shift_eval(p, R.0, x) = polynomial_eval(p, x + R.0)
    x + R.0 = x
    polynomial_eval(p, x + R.0) = polynomial_eval(p, x)
    polynomial_shift_eval(p, R.0, x) = polynomial_eval(p, x)
}

/// Translating by `a` and then by `b` is translating by `a + b`.
theorem polynomial_shift_eval_twice[R: CommRing](p: Polynomial[R], a: R, b: R, x: R) {
    polynomial_eval(p, (x + a) + b) = polynomial_shift_eval(p, a + b, x)
} by {
    polynomial_shift_eval(p, a + b, x) = polynomial_eval(p, x + (a + b))
    (x + a) + b = x + (a + b)
    polynomial_eval(p, (x + a) + b) = polynomial_eval(p, x + (a + b))
    polynomial_eval(p, (x + a) + b) = polynomial_shift_eval(p, a + b, x)
}

/// Translating a constant polynomial changes no value.
theorem polynomial_shift_eval_constant[R: CommRing](r: R, a: R, x: R) {
    polynomial_shift_eval(Polynomial[R].constant(r), a, x) = r
} by {
    polynomial_shift_eval(Polynomial[R].constant(r), a, x) =
        polynomial_eval(Polynomial[R].constant(r), x + a)
    polynomial_eval_constant(r, x + a)
    polynomial_eval(Polynomial[R].constant(r), x + a) = r
    polynomial_shift_eval(Polynomial[R].constant(r), a, x) = r
}

/// The concrete shift of a linear polynomial: the linear polynomial
/// `a * X + (b + a * t)` evaluates at `x` as the original linear polynomial
/// evaluated at `x + t`.
theorem linear_polynomial_shift_eval[R: CommRing](a: R, b: R, t: R, x: R) {
    polynomial_eval(linear_polynomial(a, b + a * t), x) =
        polynomial_eval(linear_polynomial(a, b), x + t)
} by {
    linear_polynomial_eval(a, b + a * t, x)
    polynomial_eval(linear_polynomial(a, b + a * t), x) = (b + a * t) + a * x
    linear_polynomial_eval(a, b, x + t)
    polynomial_eval(linear_polynomial(a, b), x + t) = b + a * (x + t)
    a * (x + t) = a * x + a * t
    b + a * (x + t) = b + (a * x + a * t)
    b + (a * x + a * t) = (b + a * t) + a * x
    (b + a * t) + a * x = b + a * (x + t)
    polynomial_eval(linear_polynomial(a, b + a * t), x) =
        polynomial_eval(linear_polynomial(a, b), x + t)
}

// ---------------------------------------------------------------------------
// Degrees
// ---------------------------------------------------------------------------

/// The zero polynomial has degree at most any natural number.
theorem polynomial_degree_le_zero[R: Semiring](d: Nat) {
    polynomial_degree_le(Polynomial[R].zero, d)
} by {
    polynomial_degree_le(Polynomial[R].zero, d) = polynomial_support_bounded_by(Polynomial[R].zero, d.suc)
    polynomial_zero_support_bounded_by_zero[R]
    polynomial_support_bounded_by(Polynomial[R].zero, Nat.0)
    Nat.0 <= d.suc
    polynomial_support_bounded_by_monotone(Polynomial[R].zero, Nat.0, d.suc)
    polynomial_support_bounded_by(Polynomial[R].zero, d.suc)
    polynomial_degree_le(Polynomial[R].zero, d)
}

/// A constant polynomial has degree at most zero.
theorem polynomial_degree_constant_le_zero[R: Semiring](r: R) {
    polynomial_degree_le(Polynomial[R].constant(r), Nat.0)
} by {
    polynomial_degree_le(Polynomial[R].constant(r), Nat.0) =
        polynomial_support_bounded_by(Polynomial[R].constant(r), Nat.0.suc)
    polynomial_constant_support_bounded_by_one(r)
    polynomial_support_bounded_by(Polynomial[R].constant(r), Nat.0.suc)
    polynomial_degree_le(Polynomial[R].constant(r), Nat.0)
}

/// A monomial of exponent `n` has degree at most `n`.
theorem polynomial_degree_monomial_le[R: Semiring](n: Nat, r: R) {
    polynomial_degree_le(Polynomial[R].monomial(n, r), n)
} by {
    polynomial_degree_le(Polynomial[R].monomial(n, r), n) =
        polynomial_support_bounded_by(Polynomial[R].monomial(n, r), n.suc)
    polynomial_monomial_support_bounded_by_suc(n, r)
    polynomial_support_bounded_by(Polynomial[R].monomial(n, r), n.suc)
    polynomial_degree_le(Polynomial[R].monomial(n, r), n)
}

/// A degree bound can be raised.
theorem polynomial_degree_le_monotone[R: Semiring](p: Polynomial[R], d: Nat, e: Nat) {
    polynomial_degree_le(p, d) and d <= e implies polynomial_degree_le(p, e)
} by {
    if polynomial_degree_le(p, d) and d <= e {
        polynomial_degree_le(p, d) = polynomial_support_bounded_by(p, d.suc)
        polynomial_support_bounded_by(p, d.suc)
        lte_suc_suc(d, e)
        d.suc <= e.suc
        polynomial_support_bounded_by_monotone(p, d.suc, e.suc)
        polynomial_support_bounded_by(p, e.suc)
        polynomial_degree_le(p, e) = polynomial_support_bounded_by(p, e.suc)
        polynomial_degree_le(p, e)
    }
}

/// The classical bound for the degree of a product:
/// `deg(f * g) <= deg f + deg g + 1` in strict-support-bound terms.
theorem polynomial_degree_mul_le_add[R: Semiring](
    p: Polynomial[R],
    q: Polynomial[R],
    m: Nat,
    n: Nat
) {
    polynomial_degree_le(p, m) and polynomial_degree_le(q, n) implies
    polynomial_degree_le(polynomial_mul(p, q), m + n + Nat.1)
} by {
    if polynomial_degree_le(p, m) and polynomial_degree_le(q, n) {
        polynomial_degree_le(p, m) = polynomial_support_bounded_by(p, m.suc)
        polynomial_support_bounded_by(p, m.suc)
        polynomial_degree_le(q, n) = polynomial_support_bounded_by(q, n.suc)
        polynomial_support_bounded_by(q, n.suc)
        polynomial_mul_support_bounded_by_add(p, q, m.suc, n.suc)
        polynomial_support_bounded_by(polynomial_mul(p, q), m.suc + n.suc)
        (m + n + Nat.1).suc = m.suc + n.suc
        polynomial_support_bounded_by(polynomial_mul(p, q), (m + n + Nat.1).suc)
        polynomial_degree_le(polynomial_mul(p, q), m + n + Nat.1) =
            polynomial_support_bounded_by(polynomial_mul(p, q), (m + n + Nat.1).suc)
        polynomial_degree_le(polynomial_mul(p, q), m + n + Nat.1)
    }
}

// ---------------------------------------------------------------------------
// The zero polynomial
// ---------------------------------------------------------------------------

/// A partial sum whose summands all vanish within the bound is zero.
lemma theorems_deep_partial_zero_of_pointwise_zero[R: Semiring](f: Nat -> R, n: Nat) {
    (forall(i: Nat) { i < n implies f(i) = R.0 }) implies partial(f, n) = R.0
} by {
    define statement(k: Nat) -> Bool {
        forall(g: Nat -> R) {
            (forall(i: Nat) { i < k implies g(i) = R.0 }) implies partial(g, k) = R.0
        }
    }

    forall(g: Nat -> R) {
        partial_zero[R](g)
        partial(g, Nat.0) = R.0
    }
    statement(Nat.0)

    forall(k: Nat) {
        if statement(k) {
            forall(g: Nat -> R) {
                if forall(i: Nat) { i < k.suc implies g(i) = R.0 } {
                    partial_split_last[R](g, k)
                    partial(g, k.suc) = partial(g, k) + g(k)
                    forall(i: Nat) {
                        if i < k {
                            lt_imp_lt_suc(i, k)
                            i < k.suc
                            g(i) = R.0
                        }
                    }
                    statement(k) = forall(h: Nat -> R) {
                        (forall(j: Nat) { j < k implies h(j) = R.0 }) implies partial(h, k) = R.0
                    }
                    partial(g, k) = R.0
                    lt_suc(k)
                    k < k.suc
                    g(k) = R.0
                    partial(g, k.suc) = R.0 + R.0
                    R.0 + R.0 = R.0
                    partial(g, k.suc) = R.0
                }
            }
            statement(k.suc)
        }
    }

    statement(Nat.0) and forall(k: Nat) { statement(k) implies statement(k.suc) }
    alt_induction(statement)
    forall(k: Nat) { statement(k) }
    statement(n)
    statement(n) = forall(g: Nat -> R) {
        (forall(i: Nat) { i < n implies g(i) = R.0 }) implies partial(g, n) = R.0
    }
    partial(f, n) = R.0
}

/// A polynomial is zero exactly when every coefficient is zero.
theorem polynomial_eq_zero_iff_coeff_zero[R: Semiring](p: Polynomial[R]) {
    (p = Polynomial[R].zero) iff (forall(n: Nat) { p.coeff(n) = R.0 })
} by {
    if p = Polynomial[R].zero {
        forall(n: Nat) {
            polynomial_coeff_zero_of_eq_zero(p, n)
            p.coeff(n) = R.0
        }
    }
    if forall(n: Nat) { p.coeff(n) = R.0 } {
        polynomial_eq_zero_of_coeff_zero(p)
        p = Polynomial[R].zero
    }
}

/// A polynomial whose coefficients all vanish evaluates to zero everywhere.
theorem polynomial_eval_of_coeff_zero[R: Semiring](p: Polynomial[R], x: R) {
    (forall(n: Nat) { p.coeff(n) = R.0 }) implies polynomial_eval(p, x) = R.0
} by {
    if forall(n: Nat) { p.coeff(n) = R.0 } {
        polynomial_eq_zero_of_coeff_zero(p)
        p = Polynomial[R].zero
        polynomial_eval_eq_of_eq(p, Polynomial[R].zero, x)
        polynomial_eval(p, x) = polynomial_eval(Polynomial[R].zero, x)
        polynomial_eval_zero(x)
        polynomial_eval(Polynomial[R].zero, x) = R.0
        polynomial_eval(p, x) = R.0
    }
}

/// The constant polynomial with constant zero is the zero polynomial, so its
/// evaluation vanishes.
theorem polynomial_eval_constant_zero[R: Semiring](x: R) {
    polynomial_eval(Polynomial[R].constant(R.0), x) = R.0
} by {
    polynomial_eval_constant(R.0, x)
    polynomial_eval(Polynomial[R].constant(R.0), x) = R.0
}

/// The zero polynomial absorbs products on the left.
theorem polynomial_mul_zero_left[R: Semiring](p: Polynomial[R]) {
    polynomial_mul(Polynomial[R].zero, p) = Polynomial[R].zero
} by {
    forall(k: Nat) {
        let term_fn: Nat -> R = function(i: Nat) {
            polynomial_mul_term_coeff(Polynomial[R].zero, p, k, i)
        }
        forall(i: Nat) {
            if i < k.suc {
                term_fn(i) = polynomial_mul_term_coeff(Polynomial[R].zero, p, k, i)
                polynomial_mul_term_coeff(Polynomial[R].zero, p, k, i) =
                    Polynomial[R].zero.coeff(i) * p.coeff(k - i)
                polynomial_zero_coeff[R](i)
                Polynomial[R].zero.coeff(i) = R.0
                polynomial_mul_term_coeff(Polynomial[R].zero, p, k, i) = R.0 * p.coeff(k - i)
                R.0 * p.coeff(k - i) = R.0
                term_fn(i) = R.0
            }
        }
        theorems_deep_partial_zero_of_pointwise_zero(term_fn, k.suc)
        partial(term_fn, k.suc) = R.0
        polynomial_mul_coeff_apply(Polynomial[R].zero, p, k)
        polynomial_mul(Polynomial[R].zero, p).coeff(k) = polynomial_mul_coeff(Polynomial[R].zero, p, k)
        polynomial_mul_coeff(Polynomial[R].zero, p, k) =
            partial(function(i: Nat) { polynomial_mul_term_coeff(Polynomial[R].zero, p, k, i) }, k.suc)
        polynomial_mul_coeff(Polynomial[R].zero, p, k) = partial(term_fn, k.suc)
        polynomial_mul(Polynomial[R].zero, p).coeff(k) = R.0
        polynomial_zero_coeff[R](k)
        polynomial_mul(Polynomial[R].zero, p).coeff(k) = Polynomial[R].zero.coeff(k)
    }
    polynomial_ext_pointwise(polynomial_mul(Polynomial[R].zero, p), Polynomial[R].zero)
}

/// The zero polynomial absorbs products on the right.
theorem polynomial_mul_zero_right[R: Semiring](p: Polynomial[R]) {
    polynomial_mul(p, Polynomial[R].zero) = Polynomial[R].zero
} by {
    forall(k: Nat) {
        let term_fn: Nat -> R = function(i: Nat) {
            polynomial_mul_term_coeff(p, Polynomial[R].zero, k, i)
        }
        forall(i: Nat) {
            if i < k.suc {
                term_fn(i) = polynomial_mul_term_coeff(p, Polynomial[R].zero, k, i)
                polynomial_mul_term_coeff(p, Polynomial[R].zero, k, i) =
                    p.coeff(i) * Polynomial[R].zero.coeff(k - i)
                polynomial_zero_coeff[R](k - i)
                Polynomial[R].zero.coeff(k - i) = R.0
                polynomial_mul_term_coeff(p, Polynomial[R].zero, k, i) = p.coeff(i) * R.0
                p.coeff(i) * R.0 = R.0
                term_fn(i) = R.0
            }
        }
        theorems_deep_partial_zero_of_pointwise_zero(term_fn, k.suc)
        partial(term_fn, k.suc) = R.0
        polynomial_mul_coeff_apply(p, Polynomial[R].zero, k)
        polynomial_mul(p, Polynomial[R].zero).coeff(k) = polynomial_mul_coeff(p, Polynomial[R].zero, k)
        polynomial_mul_coeff(p, Polynomial[R].zero, k) =
            partial(function(i: Nat) { polynomial_mul_term_coeff(p, Polynomial[R].zero, k, i) }, k.suc)
        polynomial_mul_coeff(p, Polynomial[R].zero, k) = partial(term_fn, k.suc)
        polynomial_mul(p, Polynomial[R].zero).coeff(k) = R.0
        polynomial_zero_coeff[R](k)
        polynomial_mul(p, Polynomial[R].zero).coeff(k) = Polynomial[R].zero.coeff(k)
    }
    polynomial_ext_pointwise(polynomial_mul(p, Polynomial[R].zero), Polynomial[R].zero)
}

/// A partial sum in which every summand after the first vanishes equals the
/// first summand.
lemma polynomial_partial_first_only[R: Semiring](f: Nat -> R, n: Nat) {
    (forall(i: Nat) { i < n.suc implies (i != Nat.0 implies f(i) = R.0) }) implies
    partial(f, n.suc) = f(Nat.0)
} by {
    define statement(m: Nat) -> Bool {
        (forall(i: Nat) { i < m.suc implies (i != Nat.0 implies f(i) = R.0) }) implies
        partial(f, m.suc) = f(Nat.0)
    }

    if forall(i: Nat) { i < Nat.0.suc implies (i != Nat.0 implies f(i) = R.0) } {
        partial_one[R](f)
        partial(f, Nat.0.suc) = f(Nat.0)
    }
    statement(Nat.0)

    forall(m: Nat) {
        if statement(m) {
            if forall(i: Nat) { i < m.suc.suc implies (i != Nat.0 implies f(i) = R.0) } {
                partial_split_last[R](f, m.suc)
                partial(f, m.suc.suc) = partial(f, m.suc) + f(m.suc)
                forall(i: Nat) {
                    if i < m.suc {
                        lt_imp_lt_suc(i, m.suc)
                        i < m.suc.suc
                        if i != Nat.0 {
                            f(i) = R.0
                        }
                    }
                }
                statement(m) =
                    ((forall(i: Nat) { i < m.suc implies (i != Nat.0 implies f(i) = R.0) }) implies
                    partial(f, m.suc) = f(Nat.0))
                partial(f, m.suc) = f(Nat.0)
                lt_suc(m.suc)
                m.suc < m.suc.suc
                alt_suc_ne_zero(m)
                m.suc != Nat.0
                f(m.suc) = R.0
                partial(f, m.suc.suc) = f(Nat.0) + R.0
                f(Nat.0) + R.0 = f(Nat.0)
                partial(f, m.suc.suc) = f(Nat.0)
            }
            statement(m.suc)
        }
    }

    statement(Nat.0) and forall(m: Nat) { statement(m) implies statement(m.suc) }
    alt_induction(statement)
    forall(m: Nat) { statement(m) }
    if forall(i: Nat) { i < n.suc implies (i != Nat.0 implies f(i) = R.0) } {
        statement(n)
        statement(n) = ((forall(i: Nat) { i < n.suc implies (i != Nat.0 implies f(i) = R.0) }) implies
            partial(f, n.suc) = f(Nat.0))
        partial(f, n.suc) = f(Nat.0)
    }
}

/// The one polynomial is a left identity for multiplication.
theorem polynomial_mul_one_left[R: Semiring](p: Polynomial[R]) {
    polynomial_mul(Polynomial[R].one, p) = p
} by {
    forall(k: Nat) {
        let term_fn: Nat -> R = function(i: Nat) {
            polynomial_mul_term_coeff(Polynomial[R].one, p, k, i)
        }
        forall(i: Nat) {
            if i < k.suc {
                if i != Nat.0 {
                    term_fn(i) = polynomial_mul_term_coeff(Polynomial[R].one, p, k, i)
                    polynomial_mul_term_coeff(Polynomial[R].one, p, k, i) =
                        Polynomial[R].one.coeff(i) * p.coeff(k - i)
                    polynomial_one_coeff_of_ne_zero[R](i)
                    Polynomial[R].one.coeff(i) = R.0
                    polynomial_mul_term_coeff(Polynomial[R].one, p, k, i) = R.0 * p.coeff(k - i)
                    R.0 * p.coeff(k - i) = R.0
                    term_fn(i) = R.0
                }
            }
        }
        polynomial_partial_first_only(term_fn, k)
        partial(term_fn, k.suc) = term_fn(Nat.0)
        term_fn(Nat.0) = polynomial_mul_term_coeff(Polynomial[R].one, p, k, Nat.0)
        polynomial_mul_term_coeff(Polynomial[R].one, p, k, Nat.0) =
            Polynomial[R].one.coeff(Nat.0) * p.coeff(k - Nat.0)
        polynomial_one_coeff_zero[R]
        Polynomial[R].one.coeff(Nat.0) = R.1
        k - Nat.0 = k
        polynomial_mul_term_coeff(Polynomial[R].one, p, k, Nat.0) = R.1 * p.coeff(k)
        R.1 * p.coeff(k) = p.coeff(k)
        term_fn(Nat.0) = p.coeff(k)
        partial(term_fn, k.suc) = p.coeff(k)
        polynomial_mul_coeff_apply(Polynomial[R].one, p, k)
        polynomial_mul(Polynomial[R].one, p).coeff(k) = polynomial_mul_coeff(Polynomial[R].one, p, k)
        polynomial_mul_coeff(Polynomial[R].one, p, k) =
            partial(function(i: Nat) { polynomial_mul_term_coeff(Polynomial[R].one, p, k, i) }, k.suc)
        polynomial_mul_coeff(Polynomial[R].one, p, k) = partial(term_fn, k.suc)
        polynomial_mul(Polynomial[R].one, p).coeff(k) = p.coeff(k)
    }
    polynomial_ext_pointwise(polynomial_mul(Polynomial[R].one, p), p)
}

/// The one polynomial is a right identity for multiplication.
theorem polynomial_mul_one_right[R: Semiring](p: Polynomial[R]) {
    polynomial_mul(p, Polynomial[R].one) = p
} by {
    forall(k: Nat) {
        let term_fn: Nat -> R = function(i: Nat) {
            polynomial_mul_term_coeff(p, Polynomial[R].one, k, i)
        }
        forall(i: Nat) {
            if i < k {
                term_fn(i) = polynomial_mul_term_coeff(p, Polynomial[R].one, k, i)
                polynomial_mul_term_coeff(p, Polynomial[R].one, k, i) =
                    p.coeff(i) * Polynomial[R].one.coeff(k - i)
                sub_pos(k, i)
                k - i > Nat.0
                k - i != Nat.0
                polynomial_one_coeff_of_ne_zero[R](k - i)
                Polynomial[R].one.coeff(k - i) = R.0
                polynomial_mul_term_coeff(p, Polynomial[R].one, k, i) = p.coeff(i) * R.0
                p.coeff(i) * R.0 = R.0
                term_fn(i) = R.0
            }
        }
        theorems_deep_partial_zero_of_pointwise_zero(term_fn, k)
        partial(term_fn, k) = R.0
        partial_split_last[R](term_fn, k)
        partial(term_fn, k.suc) = partial(term_fn, k) + term_fn(k)
        partial(term_fn, k.suc) = R.0 + term_fn(k)
        R.0 + term_fn(k) = term_fn(k)
        term_fn(k) = polynomial_mul_term_coeff(p, Polynomial[R].one, k, k)
        polynomial_mul_term_coeff(p, Polynomial[R].one, k, k) =
            p.coeff(k) * Polynomial[R].one.coeff(k - k)
        sub_self(k)
        k - k = Nat.0
        polynomial_one_coeff_zero[R]
        Polynomial[R].one.coeff(Nat.0) = R.1
        polynomial_mul_term_coeff(p, Polynomial[R].one, k, k) = p.coeff(k) * R.1
        p.coeff(k) * R.1 = p.coeff(k)
        term_fn(k) = p.coeff(k)
        partial(term_fn, k.suc) = p.coeff(k)
        polynomial_mul_coeff_apply(p, Polynomial[R].one, k)
        polynomial_mul(p, Polynomial[R].one).coeff(k) = polynomial_mul_coeff(p, Polynomial[R].one, k)
        polynomial_mul_coeff(p, Polynomial[R].one, k) =
            partial(function(i: Nat) { polynomial_mul_term_coeff(p, Polynomial[R].one, k, i) }, k.suc)
        polynomial_mul_coeff(p, Polynomial[R].one, k) = partial(term_fn, k.suc)
        polynomial_mul(p, Polynomial[R].one).coeff(k) = p.coeff(k)
    }
    polynomial_ext_pointwise(polynomial_mul(p, Polynomial[R].one), p)
}

// ---------------------------------------------------------------------------
// Deferred theorems
// ---------------------------------------------------------------------------

/// Full associativity of polynomial multiplication at the coefficient level:
/// `(p * q) * r = p * (q * r)`.
///
/// Deferred: the coefficient of `X^k` in both iterated products is a double
/// convolution sum, and proving the reindexing of the inner sums requires a
/// substantial development of range-sum reindexing lemmas over `partial`.
/// The evaluation-level associativity is proved above
/// (`polynomial_eval_mul_assoc`).
// theorem polynomial_mul_assoc[R: CommRing](p: Polynomial[R], q: Polynomial[R], r: Polynomial[R]) {
//     polynomial_mul(polynomial_mul(p, q), r) = polynomial_mul(p, polynomial_mul(q, r))
// }

/// Equal evaluations at every point force polynomial equality.
///
/// Deferred and false in general: over the two-element field the polynomial
/// `X^2 - X` evaluates to zero at every point without being the zero
/// polynomial. A correct statement needs a field with infinitely many
/// elements, which is not yet available in this library.
// theorem polynomial_eq_of_eval_eq[F: Field](p: Polynomial[F], q: Polynomial[F]) {
//     (forall(x: F) { polynomial_eval(p, x) = polynomial_eval(q, x) }) implies p = q
// }
