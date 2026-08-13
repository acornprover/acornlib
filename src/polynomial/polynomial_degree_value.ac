/// A degree-valued function for polynomials.
///
/// The library represents degrees through the predicate
/// `polynomial_degree_le(p, d)`.  This file adds a genuine degree function
/// `polynomial_degree(p)` returning the exact degree of a nonzero polynomial
/// (and zero for the zero polynomial), together with the basic laws that make
/// the function usable.

from nat import Nat, alt_induction, lt_suc, not_lt_zero, trichotomy
from semiring import Semiring
from order import lt_trans
from polynomial import Polynomial, polynomial_constant, polynomial_monomial,
    polynomial_zero_coeff, polynomial_ext_pointwise, polynomial_monomial_coeff_self,
    polynomial_support_bounded_by, polynomial_support_bounded_by_apply,
    polynomial_support_bound_exists, polynomial_eq_zero_of_coeff_zero
from polynomial.polynomial_degree import polynomial_degree_le, polynomial_degree_le_monotone,
    polynomial_coeff_zero_of_degree_lt, polynomial_degree_drop_top, polynomial_degree_le_zero,
    polynomial_support_bounded_by_drop_top, polynomial_degree_monomial_le

numerals Nat

/// A support-bounded nonzero polynomial has an exact degree below the bound:
/// a largest exponent with a nonzero coefficient.
define degree_bound_at[R: Semiring](p: Polynomial[R], n: Nat) -> Bool {
    polynomial_support_bounded_by(p, n) implies
    (p = Polynomial[R].zero or
        exists(d: Nat) {
            d < n and polynomial_degree_le(p, d) and p.coeff(d) != R.0
        })
}

/// A support-bounded nonzero polynomial has an exact degree below the bound.
theorem polynomial_exact_degree_bound[R: Semiring](p: Polynomial[R], n: Nat) {
    polynomial_support_bounded_by(p, n) implies
    (p = Polynomial[R].zero or
        exists(d: Nat) {
            d < n and polynomial_degree_le(p, d) and p.coeff(d) != R.0
        })
} by {
    define statement(m: Nat) -> Bool {
        forall(p0: Polynomial[R]) {
            degree_bound_at(p0, m)
        }
    }

    forall(p0: Polynomial[R]) {
        if polynomial_support_bounded_by(p0, Nat.0) {
            forall(k: Nat) {
                not_lt_zero(k)
                polynomial_support_bounded_by_apply(p0, Nat.0, k)
                p0.coeff(k) = R.0
            }
            polynomial_eq_zero_of_coeff_zero(p0)
            p0 = Polynomial[R].zero
            degree_bound_at(p0, Nat.0) = (polynomial_support_bounded_by(p0, Nat.0) implies (p0 = Polynomial[R].zero or exists(d: Nat) { d < Nat.0 and polynomial_degree_le(p0, d) and p0.coeff(d) != R.0 }))
            degree_bound_at(p0, Nat.0)
        }
    }
    forall(p0: Polynomial[R]) {
        degree_bound_at(p0, Nat.0) = (polynomial_support_bounded_by(p0, Nat.0) implies (p0 = Polynomial[R].zero or exists(d: Nat) { d < Nat.0 and polynomial_degree_le(p0, d) and p0.coeff(d) != R.0 }))
        degree_bound_at(p0, Nat.0)
    }
    statement(Nat.0) = forall(p0: Polynomial[R]) { degree_bound_at(p0, Nat.0) }
    statement(Nat.0)

    forall(m: Nat) {
        if statement(m) {
            forall(p0: Polynomial[R]) {
                if polynomial_support_bounded_by(p0, m.suc) {
                    if p0 = Polynomial[R].zero {
                        degree_bound_at(p0, m.suc)
                    } else {
                        p0 != Polynomial[R].zero
                        if p0.coeff(m) != R.0 {
                            polynomial_degree_le(p0, m) = polynomial_support_bounded_by(p0, m.suc)
                            polynomial_degree_le(p0, m)
                            lt_suc(m)
                            m < m.suc
                            exists(d: Nat) {
                                d = m and d < m.suc and polynomial_degree_le(p0, d) and p0.coeff(d) != R.0
                            }
                            exists(d: Nat) {
                                d < m.suc and polynomial_degree_le(p0, d) and p0.coeff(d) != R.0
                            }
                            degree_bound_at(p0, m.suc) = (polynomial_support_bounded_by(p0, m.suc) implies (p0 = Polynomial[R].zero or exists(d: Nat) { d < m.suc and polynomial_degree_le(p0, d) and p0.coeff(d) != R.0 }))
                            degree_bound_at(p0, m.suc)
                        } else {
                            p0.coeff(m) = R.0
                            polynomial_support_bounded_by_drop_top(p0, m)
                            polynomial_support_bounded_by(p0, m)
                            statement(m) = forall(p1: Polynomial[R]) { degree_bound_at(p1, m) }
                            degree_bound_at(p0, m)
                            degree_bound_at(p0, m) = (polynomial_support_bounded_by(p0, m) implies (p0 = Polynomial[R].zero or exists(d: Nat) { d < m and polynomial_degree_le(p0, d) and p0.coeff(d) != R.0 }))
                            p0 = Polynomial[R].zero or exists(d: Nat) { d < m and polynomial_degree_le(p0, d) and p0.coeff(d) != R.0 }
                            if p0 = Polynomial[R].zero {
                                degree_bound_at(p0, m.suc)
                            } else {
                                exists(d: Nat) { d < m and polynomial_degree_le(p0, d) and p0.coeff(d) != R.0 }
                                let d0: Nat satisfy {
                                    d0 < m and polynomial_degree_le(p0, d0) and p0.coeff(d0) != R.0
                                }
                                lt_suc(m)
                                m < m.suc
                                lt_trans[Nat](d0, m, m.suc)
                                d0 < m.suc
                                exists(d: Nat) {
                                    d = d0 and d < m.suc and polynomial_degree_le(p0, d) and p0.coeff(d) != R.0
                                }
                                exists(d: Nat) {
                                    d < m.suc and polynomial_degree_le(p0, d) and p0.coeff(d) != R.0
                                }
                                degree_bound_at(p0, m.suc)
                            }
                        }
                    }
                }
            }
            forall(p0: Polynomial[R]) {
                degree_bound_at(p0, m.suc) = (polynomial_support_bounded_by(p0, m.suc) implies (p0 = Polynomial[R].zero or exists(d: Nat) { d < m.suc and polynomial_degree_le(p0, d) and p0.coeff(d) != R.0 }))
                degree_bound_at(p0, m.suc)
            }
            statement(m.suc) = forall(p0: Polynomial[R]) { degree_bound_at(p0, m.suc) }
            statement(m.suc)
        }
    }

    statement(Nat.0) and forall(m: Nat) { statement(m) implies statement(m.suc) }
    alt_induction(statement)
    forall(m: Nat) { statement(m) }
    statement(n)
    statement(n) = forall(p0: Polynomial[R]) { degree_bound_at(p0, n) }
    degree_bound_at(p, n)
    degree_bound_at(p, n) = (polynomial_support_bounded_by(p, n) implies (p = Polynomial[R].zero or exists(d: Nat) { d < n and polynomial_degree_le(p, d) and p.coeff(d) != R.0 }))
    polynomial_support_bounded_by(p, n)
    p = Polynomial[R].zero or exists(d: Nat) { d < n and polynomial_degree_le(p, d) and p.coeff(d) != R.0 }
}

/// A nonzero polynomial has a largest exponent with a nonzero coefficient.
theorem polynomial_degree_exists[R: Semiring](p: Polynomial[R]) {
    exists(result: Nat) {
        (p = Polynomial[R].zero implies result = Nat.0) and
        (p != Polynomial[R].zero implies polynomial_degree_le(p, result) and p.coeff(result) != R.0)
    }
} by {
    if p = Polynomial[R].zero {
        exists(result0: Nat) {
            result0 = Nat.0 and (p = Polynomial[R].zero implies result0 = Nat.0) and (p != Polynomial[R].zero implies polynomial_degree_le(p, result0) and p.coeff(result0) != R.0)
        }
        exists(result: Nat) {
            (p = Polynomial[R].zero implies result = Nat.0) and (p != Polynomial[R].zero implies polynomial_degree_le(p, result) and p.coeff(result) != R.0)
        }
    } else {
        p != Polynomial[R].zero
        polynomial_support_bound_exists(p)
        let n: Nat satisfy {
            polynomial_support_bounded_by(p, n)
        }
        polynomial_exact_degree_bound(p, n)
        p = Polynomial[R].zero or exists(d: Nat) { d < n and polynomial_degree_le(p, d) and p.coeff(d) != R.0 }
        exists(d: Nat) { d < n and polynomial_degree_le(p, d) and p.coeff(d) != R.0 }
        let d0: Nat satisfy {
            d0 < n and polynomial_degree_le(p, d0) and p.coeff(d0) != R.0
        }
        exists(result0: Nat) {
            result0 = d0 and (p = Polynomial[R].zero implies result0 = Nat.0) and (p != Polynomial[R].zero implies polynomial_degree_le(p, result0) and p.coeff(result0) != R.0)
        }
        exists(result: Nat) {
            (p = Polynomial[R].zero implies result = Nat.0) and (p != Polynomial[R].zero implies polynomial_degree_le(p, result) and p.coeff(result) != R.0)
        }
    }
    exists(result: Nat) {
        (p = Polynomial[R].zero implies result = Nat.0) and (p != Polynomial[R].zero implies polynomial_degree_le(p, result) and p.coeff(result) != R.0)
    }
}

/// The exact degree of a polynomial: zero for the zero polynomial, and the
/// largest exponent with a nonzero coefficient otherwise.
let polynomial_degree[R: Semiring](p: Polynomial[R]) -> result: Nat satisfy {
    (p = Polynomial[R].zero implies result = Nat.0) and
    (p != Polynomial[R].zero implies polynomial_degree_le(p, result) and p.coeff(result) != R.0)
} by {
    polynomial_degree_exists(p)
}

/// The degree function is an upper bound for the degree of a polynomial.
theorem polynomial_degree_le_self[R: Semiring](p: Polynomial[R]) {
    polynomial_degree_le(p, polynomial_degree(p))
} by {
    if p = Polynomial[R].zero {
        polynomial_degree(p) = Nat.0
        polynomial_degree_le_zero[R](polynomial_degree(p))
        polynomial_degree_le(Polynomial[R].zero, polynomial_degree(p))
        p = Polynomial[R].zero
        polynomial_degree_le(p, polynomial_degree(p))
    } else {
        (p = Polynomial[R].zero implies polynomial_degree(p) = Nat.0) and (p != Polynomial[R].zero implies polynomial_degree_le(p, polynomial_degree(p)) and p.coeff(polynomial_degree(p)) != R.0)
        polynomial_degree_le(p, polynomial_degree(p)) and p.coeff(polynomial_degree(p)) != R.0
        polynomial_degree_le(p, polynomial_degree(p))
    }
}

/// The degree of the zero polynomial is zero.
theorem polynomial_degree_zero[R: Semiring] {
    polynomial_degree(Polynomial[R].zero) = Nat.0
} by {
    Polynomial[R].zero = Polynomial[R].zero
    polynomial_degree(Polynomial[R].zero) = Nat.0
}

/// A nonzero polynomial has a nonzero coefficient at its degree.
theorem polynomial_degree_coeff_nonzero[R: Semiring](p: Polynomial[R]) {
    p != Polynomial[R].zero implies p.coeff(polynomial_degree(p)) != R.0
} by {
    if p != Polynomial[R].zero {
        (p = Polynomial[R].zero implies polynomial_degree(p) = Nat.0) and (p != Polynomial[R].zero implies polynomial_degree_le(p, polynomial_degree(p)) and p.coeff(polynomial_degree(p)) != R.0)
        polynomial_degree_le(p, polynomial_degree(p)) and p.coeff(polynomial_degree(p)) != R.0
        p.coeff(polynomial_degree(p)) != R.0
    }
}

/// A degree bound attained by a nonzero coefficient is the exact degree.
theorem polynomial_degree_exact[R: Semiring](p: Polynomial[R], d: Nat) {
    polynomial_degree_le(p, d) and p.coeff(d) != R.0 implies polynomial_degree(p) = d
} by {
    if polynomial_degree_le(p, d) and p.coeff(d) != R.0 {
        if p = Polynomial[R].zero {
            polynomial_zero_coeff[R](d)
            Polynomial[R].zero.coeff(d) = R.0
            p.coeff(d) = R.0
            false
        }
        p != Polynomial[R].zero
        (p = Polynomial[R].zero implies polynomial_degree(p) = Nat.0) and (p != Polynomial[R].zero implies polynomial_degree_le(p, polynomial_degree(p)) and p.coeff(polynomial_degree(p)) != R.0)
        polynomial_degree_le(p, polynomial_degree(p)) and p.coeff(polynomial_degree(p)) != R.0
        trichotomy(d, polynomial_degree(p))
        if d < polynomial_degree(p) {
            polynomial_coeff_zero_of_degree_lt(p, d, polynomial_degree(p))
            p.coeff(polynomial_degree(p)) = R.0
            false
        }
        if polynomial_degree(p) < d {
            polynomial_coeff_zero_of_degree_lt(p, polynomial_degree(p), d)
            p.coeff(d) = R.0
            false
        }
        d = polynomial_degree(p)
        polynomial_degree(p) = d
    }
}

/// The degree of a monomial with a nonzero coefficient is its exponent.
theorem polynomial_degree_monomial[R: Semiring](n: Nat, r: R) {
    r != R.0 implies polynomial_degree(polynomial_monomial(n, r)) = n
} by {
    if r != R.0 {
        polynomial_degree_monomial_le(n, r)
        polynomial_degree_le(polynomial_monomial(n, r), n)
        polynomial_monomial_coeff_self(n, r)
        polynomial_monomial(n, r).coeff(n) = r
        r != R.0
        polynomial_monomial(n, r).coeff(n) != R.0
        polynomial_degree_exact(polynomial_monomial(n, r), n)
        polynomial_degree(polynomial_monomial(n, r)) = n
    }
}
