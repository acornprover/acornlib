/// Evaluation of polynomial products.

from comm_ring import CommRing
from algebra.comm_semigroup import CommSemigroup
from data.basic.function_algebra import pointwise_add
from data.basic.functions import compose, function_extensionality
from list import partial, partial_one, partial_pointwise_eq, partial_shift_suc, partial_split_last, partial_zero
from nat import Nat, add_sub, add_suc_left, alt_induction, lt_cancel_suc, lt_imp_lte_suc,
    lt_suc, lt_trans, not_lt_zero
from polynomial.add_monoid_algebra_bridge import coeff_eval_eq_of_zero_from_le,
    coeff_eval_stable_suc_of_zero_from, coeff_zero_at, coeff_zero_from,
    coeff_zero_from_apply, coeff_zero_from_at
from polynomial.base import Polynomial
from polynomial.coeff_eval import coeff_eval, coeff_eval_add, coeff_tail
from polynomial.eval import polynomial_eval_bound, polynomial_eval_bound_eq_coeff_eval,
    polynomial_support_bounded_by, polynomial_support_bound_exists
from polynomial.global_eval import polynomial_eval, polynomial_eval_eq_eval_bound_of_support_bounded
from polynomial.mul import polynomial_mul, polynomial_mul_coeff, polynomial_mul_coeff_eq,
    polynomial_mul_support_bounded_by_add
from algebra.semigroup import Semigroup

numerals Nat

lemma eval_mul_coeff_zero_from_at_intro[R: CommRing](c: Nat -> R, n: Nat, k: Nat) {
    not k < n and c(k) = R.0 implies coeff_zero_from_at(c, n, k)
} by {
    if not k < n and c(k) = R.0 {
        coeff_zero_at(c, k)
        coeff_zero_from_at(c, n, k) = (not k < n implies coeff_zero_at(c, k))
        coeff_zero_from_at(c, n, k)
    }
}

lemma eval_mul_coeff_zero_from_intro[R: CommRing](c: Nat -> R, n: Nat) {
    (forall(k: Nat) { coeff_zero_from_at(c, n, k) }) implies coeff_zero_from(c, n)
} by {
    forall(k: Nat) {
        coeff_zero_from_at(c, n, k)
    }
    coeff_zero_from(c, n) = forall(k: Nat) {
        coeff_zero_from_at(c, n, k)
    }
}

lemma eval_mul_not_suc_lt_of_not_lt(a: Nat, b: Nat) {
    not a < b implies not a.suc < b
} by {
    if not a < b {
        if a.suc < b {
            lt_suc(a)
            a < a.suc
            lt_trans(a, a.suc, b)
            a < b
            false
        }
    }
}

lemma eval_mul_not_suc_lt_suc_of_not_lt(a: Nat, b: Nat) {
    not a < b implies not a.suc < b.suc
} by {
    if not a < b {
        if a.suc < b.suc {
            lt_cancel_suc(a, b)
            a < b
            false
        }
    }
}

lemma eval_mul_coeff_zero_from_tail_same_bound[R: CommRing](c: Nat -> R, n: Nat) {
    coeff_zero_from(c, n) implies coeff_zero_from(coeff_tail(c), n)
} by {
    if coeff_zero_from(c, n) {
        forall(k: Nat) {
            if k < n {
                coeff_zero_from_at(coeff_tail(c), n, k) = (not k < n implies coeff_zero_at(coeff_tail(c), k))
                coeff_zero_from_at(coeff_tail(c), n, k)
            } else {
                not k < n
                eval_mul_not_suc_lt_of_not_lt(k, n)
                not k.suc < n
                coeff_zero_from_apply(c, n, k.suc)
                c(k.suc) = R.0
                coeff_tail(c, k) = c(k.suc)
                coeff_tail(c, k) = R.0
                coeff_zero_at(coeff_tail(c), k)
                eval_mul_coeff_zero_from_at_intro(coeff_tail(c), n, k)
                coeff_zero_from_at(coeff_tail(c), n, k)
            }
        }
        coeff_zero_from(coeff_tail(c), n) = forall(k: Nat) {
            coeff_zero_from_at(coeff_tail(c), n, k)
        }
        coeff_zero_from(coeff_tail(c), n)
    }
}

lemma eval_mul_coeff_zero_from_tail_pred[R: CommRing](c: Nat -> R, n: Nat) {
    coeff_zero_from(c, n.suc) implies coeff_zero_from(coeff_tail(c), n)
} by {
    if coeff_zero_from(c, n.suc) {
        forall(k: Nat) {
            if k < n {
                coeff_zero_from_at(coeff_tail(c), n, k) = (not k < n implies coeff_zero_at(coeff_tail(c), k))
                coeff_zero_from_at(coeff_tail(c), n, k)
            } else {
                not k < n
                eval_mul_not_suc_lt_suc_of_not_lt(k, n)
                not k.suc < n.suc
                coeff_zero_from_apply(c, n.suc, k.suc)
                c(k.suc) = R.0
                coeff_tail(c, k) = c(k.suc)
                coeff_tail(c, k) = R.0
                coeff_zero_at(coeff_tail(c), k)
                eval_mul_coeff_zero_from_at_intro(coeff_tail(c), n, k)
                coeff_zero_from_at(coeff_tail(c), n, k)
            }
        }
        coeff_zero_from(coeff_tail(c), n) = forall(k: Nat) {
            coeff_zero_from_at(coeff_tail(c), n, k)
        }
        coeff_zero_from(coeff_tail(c), n)
    }
}

/// A scalar multiple of a coefficient function.
define eval_mul_scalar_coeff[R: CommRing](a: R, c: Nat -> R, i: Nat) -> R {
    a * c(i)
}

lemma eval_mul_scalar_coeff_zero_from[R: CommRing](a: R, c: Nat -> R, n: Nat) {
    coeff_zero_from(c, n) implies coeff_zero_from(eval_mul_scalar_coeff(a, c), n)
} by {
    if coeff_zero_from(c, n) {
        forall(k: Nat) {
            if k < n {
                coeff_zero_from_at(eval_mul_scalar_coeff(a, c), n, k) =
                    (not k < n implies coeff_zero_at(eval_mul_scalar_coeff(a, c), k))
                coeff_zero_from_at(eval_mul_scalar_coeff(a, c), n, k)
            } else {
                not k < n
                coeff_zero_from_apply(c, n, k)
                c(k) = R.0
                eval_mul_scalar_coeff(a, c, k) = a * c(k)
                eval_mul_scalar_coeff(a, c, k) = a * R.0
                a * R.0 = R.0
                coeff_zero_at(eval_mul_scalar_coeff(a, c), k)
                eval_mul_coeff_zero_from_at_intro(eval_mul_scalar_coeff(a, c), n, k)
                coeff_zero_from_at(eval_mul_scalar_coeff(a, c), n, k)
            }
        }
        coeff_zero_from(eval_mul_scalar_coeff(a, c), n) = forall(k: Nat) {
            coeff_zero_from_at(eval_mul_scalar_coeff(a, c), n, k)
        }
        coeff_zero_from(eval_mul_scalar_coeff(a, c), n)
    }
}

lemma eval_mul_scalar_coeff_tail[R: CommRing](a: R, c: Nat -> R) {
    coeff_tail(eval_mul_scalar_coeff(a, c)) = eval_mul_scalar_coeff(a, coeff_tail(c))
} by {
    forall(i: Nat) {
        coeff_tail(eval_mul_scalar_coeff(a, c), i) = eval_mul_scalar_coeff(a, c, i.suc)
        eval_mul_scalar_coeff(a, c, i.suc) = a * c(i.suc)
        coeff_tail(c, i) = c(i.suc)
        eval_mul_scalar_coeff(a, coeff_tail(c), i) = a * coeff_tail(c, i)
        coeff_tail(eval_mul_scalar_coeff(a, c), i) = eval_mul_scalar_coeff(a, coeff_tail(c), i)
    }
    function_extensionality(coeff_tail(eval_mul_scalar_coeff(a, c)), eval_mul_scalar_coeff(a, coeff_tail(c)))
}

define eval_mul_scalar_coeff_eval_at[R: CommRing](a: R, c: Nat -> R, x: R, n: Nat) -> Bool {
    coeff_eval(eval_mul_scalar_coeff(a, c), x, n) = a * coeff_eval(c, x, n)
}

define eval_mul_scalar_coeff_eval_property[R: CommRing](a: R, x: R, n: Nat) -> Bool {
    forall(c: Nat -> R) {
        eval_mul_scalar_coeff_eval_at(a, c, x, n)
    }
}

lemma eval_mul_scalar_coeff_eval_property_apply[R: CommRing](a: R, x: R, n: Nat, c: Nat -> R) {
    eval_mul_scalar_coeff_eval_property(a, x, n) implies eval_mul_scalar_coeff_eval_at(a, c, x, n)
} by {
    if eval_mul_scalar_coeff_eval_property(a, x, n) {
        eval_mul_scalar_coeff_eval_property(a, x, n) = forall(d: Nat -> R) {
            eval_mul_scalar_coeff_eval_at(a, d, x, n)
        }
        eval_mul_scalar_coeff_eval_at(a, c, x, n)
    }
}

lemma eval_mul_scalar_coeff_eval_property_intro[R: CommRing](a: R, x: R, n: Nat) {
    (forall(c: Nat -> R) { eval_mul_scalar_coeff_eval_at(a, c, x, n) }) implies
    eval_mul_scalar_coeff_eval_property(a, x, n)
} by {
    forall(c: Nat -> R) {
        eval_mul_scalar_coeff_eval_at(a, c, x, n)
    }
    eval_mul_scalar_coeff_eval_property(a, x, n) = forall(c: Nat -> R) {
        eval_mul_scalar_coeff_eval_at(a, c, x, n)
    }
}

lemma eval_mul_scalar_coeff_eval[R: CommRing](a: R, c: Nat -> R, x: R, n: Nat) {
    coeff_eval(eval_mul_scalar_coeff(a, c), x, n) = a * coeff_eval(c, x, n)
} by {
    define statement(k: Nat) -> Bool {
        eval_mul_scalar_coeff_eval_property(a, x, k)
    }

    forall(c0: Nat -> R) {
        coeff_eval(eval_mul_scalar_coeff(a, c0), x, Nat.0) = R.0
        coeff_eval(c0, x, Nat.0) = R.0
        a * R.0 = R.0
        coeff_eval(eval_mul_scalar_coeff(a, c0), x, Nat.0) = a * coeff_eval(c0, x, Nat.0)
        eval_mul_scalar_coeff_eval_at(a, c0, x, Nat.0) =
            (coeff_eval(eval_mul_scalar_coeff(a, c0), x, Nat.0) = a * coeff_eval(c0, x, Nat.0))
        eval_mul_scalar_coeff_eval_at(a, c0, x, Nat.0)
    }
    eval_mul_scalar_coeff_eval_property_intro(a, x, Nat.0)
    eval_mul_scalar_coeff_eval_property(a, x, Nat.0)
    statement(Nat.0)

    forall(k: Nat) {
        if statement(k) {
            forall(c0: Nat -> R) {
                eval_mul_scalar_coeff_tail(a, c0)
                coeff_tail(eval_mul_scalar_coeff(a, c0)) = eval_mul_scalar_coeff(a, coeff_tail(c0))
                coeff_eval(eval_mul_scalar_coeff(a, c0), x, k.suc) =
                    eval_mul_scalar_coeff(a, c0, Nat.0) + x * coeff_eval(coeff_tail(eval_mul_scalar_coeff(a, c0)), x, k)
                eval_mul_scalar_coeff(a, c0, Nat.0) = a * c0(Nat.0)
                coeff_eval(eval_mul_scalar_coeff(a, c0), x, k.suc) =
                    a * c0(Nat.0) + x * coeff_eval(eval_mul_scalar_coeff(a, coeff_tail(c0)), x, k)
                statement(k) = eval_mul_scalar_coeff_eval_property(a, x, k)
                eval_mul_scalar_coeff_eval_property_apply(a, x, k, coeff_tail(c0))
                eval_mul_scalar_coeff_eval_at(a, coeff_tail(c0), x, k)
                coeff_eval(eval_mul_scalar_coeff(a, coeff_tail(c0)), x, k) = a * coeff_eval(coeff_tail(c0), x, k)
                coeff_eval(eval_mul_scalar_coeff(a, c0), x, k.suc) =
                    a * c0(Nat.0) + x * (a * coeff_eval(coeff_tail(c0), x, k))
                Semigroup.mul_associative[R](x, a, coeff_eval(coeff_tail(c0), x, k))
                x * (a * coeff_eval(coeff_tail(c0), x, k)) = x * a * coeff_eval(coeff_tail(c0), x, k)
                CommSemigroup.commutative[R](x, a)
                x * a = a * x
                x * a * coeff_eval(coeff_tail(c0), x, k) = a * x * coeff_eval(coeff_tail(c0), x, k)
                Semigroup.mul_associative[R](a, x, coeff_eval(coeff_tail(c0), x, k))
                a * (x * coeff_eval(coeff_tail(c0), x, k)) = a * x * coeff_eval(coeff_tail(c0), x, k)
                x * (a * coeff_eval(coeff_tail(c0), x, k)) = a * (x * coeff_eval(coeff_tail(c0), x, k))
                coeff_eval(eval_mul_scalar_coeff(a, c0), x, k.suc) =
                    a * c0(Nat.0) + a * (x * coeff_eval(coeff_tail(c0), x, k))
                a * c0(Nat.0) + a * (x * coeff_eval(coeff_tail(c0), x, k)) =
                    a * (c0(Nat.0) + x * coeff_eval(coeff_tail(c0), x, k))
                coeff_eval(c0, x, k.suc) = c0(Nat.0) + x * coeff_eval(coeff_tail(c0), x, k)
                coeff_eval(eval_mul_scalar_coeff(a, c0), x, k.suc) = a * coeff_eval(c0, x, k.suc)
                eval_mul_scalar_coeff_eval_at(a, c0, x, k.suc) =
                    (coeff_eval(eval_mul_scalar_coeff(a, c0), x, k.suc) = a * coeff_eval(c0, x, k.suc))
                eval_mul_scalar_coeff_eval_at(a, c0, x, k.suc)
            }
            eval_mul_scalar_coeff_eval_property_intro(a, x, k.suc)
            eval_mul_scalar_coeff_eval_property(a, x, k.suc)
            statement(k.suc)
        }
    }

    statement(Nat.0) and forall(k: Nat) { statement(k) implies statement(k.suc) }
    alt_induction(statement)
    forall(k: Nat) { statement(k) }
    statement(n)
    eval_mul_scalar_coeff_eval_property_apply(a, x, n, c)
    eval_mul_scalar_coeff_eval_at(a, c, x, n)
}

lemma eval_mul_scalar_coeff_eval_of_zero_from_le[R: CommRing](a: R, c: Nat -> R, x: R, n: Nat, m: Nat) {
    coeff_zero_from(c, n) and n <= m implies
    coeff_eval(eval_mul_scalar_coeff(a, c), x, m) = a * coeff_eval(c, x, n)
} by {
    if coeff_zero_from(c, n) and n <= m {
        eval_mul_scalar_coeff_zero_from(a, c, n)
        coeff_eval_eq_of_zero_from_le(eval_mul_scalar_coeff(a, c), x, n, m)
        coeff_eval(eval_mul_scalar_coeff(a, c), x, m) = coeff_eval(eval_mul_scalar_coeff(a, c), x, n)
        eval_mul_scalar_coeff_eval(a, c, x, n)
        coeff_eval(eval_mul_scalar_coeff(a, c), x, n) = a * coeff_eval(c, x, n)
        coeff_eval(eval_mul_scalar_coeff(a, c), x, m) = a * coeff_eval(c, x, n)
    }
}

lemma eval_mul_coeff_eval_head_tail_same_bound[R: CommRing](c: Nat -> R, x: R, n: Nat) {
    coeff_zero_from(c, n) implies coeff_eval(c, x, n) = c(Nat.0) + x * coeff_eval(coeff_tail(c), x, n)
} by {
    if coeff_zero_from(c, n) {
        match n {
            Nat.zero {
                coeff_eval(c, x, n) = R.0
                not_lt_zero(Nat.0)
                coeff_zero_from_apply(c, Nat.0, Nat.0)
                c(Nat.0) = R.0
                coeff_eval(coeff_tail(c), x, Nat.0) = R.0
                x * R.0 = R.0
                c(Nat.0) + x * coeff_eval(coeff_tail(c), x, Nat.0) = R.0 + R.0
                R.0 + R.0 = R.0
                coeff_eval(c, x, n) = c(Nat.0) + x * coeff_eval(coeff_tail(c), x, n)
            }
            Nat.suc(pred) {
                coeff_eval(c, x, n) = c(Nat.0) + x * coeff_eval(coeff_tail(c), x, pred)
                eval_mul_coeff_zero_from_tail_pred(c, pred)
                coeff_zero_from(coeff_tail(c), pred)
                coeff_eval_stable_suc_of_zero_from(coeff_tail(c), x, pred)
                coeff_eval(coeff_tail(c), x, pred.suc) = coeff_eval(coeff_tail(c), x, pred)
                n = pred.suc
                coeff_eval(coeff_tail(c), x, n) = coeff_eval(coeff_tail(c), x, pred)
                coeff_eval(c, x, n) = c(Nat.0) + x * coeff_eval(coeff_tail(c), x, n)
            }
        }
    }
}

/// Coefficient-level convolution, independent of bundled polynomials.
define coeff_convolution_term[R: CommRing](c: Nat -> R, d: Nat -> R, k: Nat, i: Nat) -> R {
    c(i) * d(k - i)
}

define coeff_convolution[R: CommRing](c: Nat -> R, d: Nat -> R, k: Nat) -> R {
    partial(function(i: Nat) { coeff_convolution_term(c, d, k, i) }, k.suc)
}

define eval_mul_zero_nat_function[R: CommRing](i: Nat) -> R {
    R.0
}

lemma eval_mul_partial_zero_nat_function[R: CommRing](n: Nat) {
    partial(eval_mul_zero_nat_function[R], n) = R.0
} by {
    define statement(k: Nat) -> Bool {
        partial(eval_mul_zero_nat_function[R], k) = R.0
    }

    partial_zero[R](eval_mul_zero_nat_function[R])
    statement(Nat.0)

    forall(k: Nat) {
        if statement(k) {
            partial_split_last[R](eval_mul_zero_nat_function[R], k)
            partial(eval_mul_zero_nat_function[R], k.suc) =
                partial(eval_mul_zero_nat_function[R], k) + eval_mul_zero_nat_function[R](k)
            eval_mul_zero_nat_function[R](k) = R.0
            partial(eval_mul_zero_nat_function[R], k) = R.0
            partial(eval_mul_zero_nat_function[R], k.suc) = R.0 + R.0
            R.0 + R.0 = R.0
            partial(eval_mul_zero_nat_function[R], k.suc) = R.0
            statement(k.suc)
        }
    }

    statement(Nat.0) and forall(k: Nat) { statement(k) implies statement(k.suc) }
    alt_induction(statement)
    forall(k: Nat) { statement(k) }
    statement(n)
}

lemma eval_mul_partial_zero_of_pointwise_zero[R: CommRing](f: Nat -> R, n: Nat) {
    (forall(i: Nat) { i < n implies f(i) = R.0 }) implies partial(f, n) = R.0
} by {
    if forall(i: Nat) { i < n implies f(i) = R.0 } {
        forall(i: Nat) {
            if i < n {
                f(i) = R.0
                eval_mul_zero_nat_function[R](i) = R.0
                f(i) = eval_mul_zero_nat_function[R](i)
            }
        }
        partial_pointwise_eq[R](f, eval_mul_zero_nat_function[R], n)
        partial(f, n) = partial(eval_mul_zero_nat_function[R], n)
        eval_mul_partial_zero_nat_function[R](n)
        partial(f, n) = R.0
    }
}

lemma eval_mul_coeff_convolution_zero_of_left_zero_from_zero[R: CommRing](c: Nat -> R, d: Nat -> R, k: Nat) {
    coeff_zero_from(c, Nat.0) implies coeff_convolution(c, d, k) = R.0
} by {
    if coeff_zero_from(c, Nat.0) {
        let term_fn: Nat -> R = function(i: Nat) { coeff_convolution_term(c, d, k, i) }
        forall(i: Nat) {
            if i < k.suc {
                not_lt_zero(i)
                coeff_zero_from_apply(c, Nat.0, i)
                c(i) = R.0
                term_fn(i) = coeff_convolution_term(c, d, k, i)
                coeff_convolution_term(c, d, k, i) = c(i) * d(k - i)
                coeff_convolution_term(c, d, k, i) = R.0 * d(k - i)
                R.0 * d(k - i) = R.0
                term_fn(i) = R.0
            }
        }
        eval_mul_partial_zero_of_pointwise_zero(term_fn, k.suc)
        partial(term_fn, k.suc) = R.0
        coeff_convolution(c, d, k) = partial(term_fn, k.suc)
        coeff_convolution(c, d, k) = R.0
    }
}

lemma eval_mul_coeff_convolution_zero_from_left_zero_from_zero[R: CommRing](c: Nat -> R, d: Nat -> R) {
    coeff_zero_from(c, Nat.0) implies coeff_zero_from(coeff_convolution(c, d), Nat.0)
} by {
    if coeff_zero_from(c, Nat.0) {
        forall(k: Nat) {
            not_lt_zero(k)
            not k < Nat.0
            eval_mul_coeff_convolution_zero_of_left_zero_from_zero(c, d, k)
            coeff_convolution(c, d, k) = R.0
            coeff_zero_at(coeff_convolution(c, d), k)
            eval_mul_coeff_zero_from_at_intro(coeff_convolution(c, d), Nat.0, k)
            coeff_zero_from_at(coeff_convolution(c, d), Nat.0, k)
        }
        coeff_zero_from(coeff_convolution(c, d), Nat.0) = forall(k: Nat) {
            coeff_zero_from_at(coeff_convolution(c, d), Nat.0, k)
        }
        coeff_zero_from(coeff_convolution(c, d), Nat.0)
    }
}

lemma eval_mul_suc_sub_suc(a: Nat, b: Nat) {
    b < a.suc implies a.suc - b.suc = a - b
} by {
    if b < a.suc {
        if b < a {
            lt_imp_lte_suc(b, a)
            b.suc <= a
            b <= a
            add_sub(a, b)
            a - b + b = a
            a - b + b.suc = (a - b + b).suc
            a.suc - b.suc = a - b
        } else {
            b = a
            a.suc - b.suc = Nat.0
            a - b = Nat.0
            a.suc - b.suc = a - b
        }
    }
}

lemma eval_mul_coeff_convolution_tail_at[R: CommRing](c: Nat -> R, d: Nat -> R, k: Nat) {
    coeff_tail(coeff_convolution(c, d), k) =
        pointwise_add(eval_mul_scalar_coeff(c(Nat.0), coeff_tail(d)), coeff_convolution(coeff_tail(c), d), k)
} by {
    let term_fn: Nat -> R = function(i: Nat) { coeff_convolution_term(c, d, k.suc, i) }
    let shifted: Nat -> R = compose(term_fn, Nat.suc)
    let tail_term: Nat -> R = function(i: Nat) { coeff_convolution_term(coeff_tail(c), d, k, i) }

    coeff_tail(coeff_convolution(c, d), k) = coeff_convolution(c, d, k.suc)
    coeff_convolution(c, d, k.suc) = partial(term_fn, k.suc.suc)
    partial_shift_suc[R](term_fn, k.suc)
    partial(term_fn, k.suc.suc) = term_fn(Nat.0) + partial(shifted, k.suc)
    term_fn(Nat.0) = coeff_convolution_term(c, d, k.suc, Nat.0)
    coeff_convolution_term(c, d, k.suc, Nat.0) = c(Nat.0) * d(k.suc - Nat.0)
    k.suc - Nat.0 = k.suc
    term_fn(Nat.0) = c(Nat.0) * d(k.suc)
    coeff_tail(d, k) = d(k.suc)
    eval_mul_scalar_coeff(c(Nat.0), coeff_tail(d), k) = c(Nat.0) * coeff_tail(d, k)
    term_fn(Nat.0) = eval_mul_scalar_coeff(c(Nat.0), coeff_tail(d), k)

    forall(i: Nat) {
        if i < k.suc {
            shifted(i) = term_fn(i.suc)
            term_fn(i.suc) = coeff_convolution_term(c, d, k.suc, i.suc)
            coeff_convolution_term(c, d, k.suc, i.suc) = c(i.suc) * d(k.suc - i.suc)
            eval_mul_suc_sub_suc(k, i)
            k.suc - i.suc = k - i
            coeff_tail(c, i) = c(i.suc)
            coeff_convolution_term(coeff_tail(c), d, k, i) = coeff_tail(c, i) * d(k - i)
            shifted(i) = coeff_convolution_term(coeff_tail(c), d, k, i)
            tail_term(i) = coeff_convolution_term(coeff_tail(c), d, k, i)
            shifted(i) = tail_term(i)
        }
    }
    partial_pointwise_eq[R](shifted, tail_term, k.suc)
    partial(shifted, k.suc) = partial(tail_term, k.suc)
    coeff_convolution(coeff_tail(c), d, k) = partial(tail_term, k.suc)
    partial(shifted, k.suc) = coeff_convolution(coeff_tail(c), d, k)
    coeff_tail(coeff_convolution(c, d), k) =
        eval_mul_scalar_coeff(c(Nat.0), coeff_tail(d), k) + coeff_convolution(coeff_tail(c), d, k)
    pointwise_add(eval_mul_scalar_coeff(c(Nat.0), coeff_tail(d)), coeff_convolution(coeff_tail(c), d), k) =
        eval_mul_scalar_coeff(c(Nat.0), coeff_tail(d), k) + coeff_convolution(coeff_tail(c), d, k)
    coeff_tail(coeff_convolution(c, d), k) =
        pointwise_add(eval_mul_scalar_coeff(c(Nat.0), coeff_tail(d)), coeff_convolution(coeff_tail(c), d), k)
}

lemma eval_mul_coeff_convolution_tail[R: CommRing](c: Nat -> R, d: Nat -> R) {
    coeff_tail(coeff_convolution(c, d)) =
        pointwise_add(eval_mul_scalar_coeff(c(Nat.0), coeff_tail(d)), coeff_convolution(coeff_tail(c), d))
} by {
    forall(k: Nat) {
        eval_mul_coeff_convolution_tail_at(c, d, k)
    }
    function_extensionality(
        coeff_tail(coeff_convolution(c, d)),
        pointwise_add(eval_mul_scalar_coeff(c(Nat.0), coeff_tail(d)), coeff_convolution(coeff_tail(c), d))
    )
}

lemma eval_mul_convolution_zero[R: CommRing](c: Nat -> R, d: Nat -> R) {
    coeff_convolution(c, d, Nat.0) = c(Nat.0) * d(Nat.0)
} by {
    let term_fn: Nat -> R = function(i: Nat) { coeff_convolution_term(c, d, Nat.0, i) }
    coeff_convolution(c, d, Nat.0) = partial(term_fn, Nat.1)
    partial_one[R](term_fn)
    partial(term_fn, Nat.1) = term_fn(Nat.0)
    term_fn(Nat.0) = coeff_convolution_term(c, d, Nat.0, Nat.0)
    coeff_convolution_term(c, d, Nat.0, Nat.0) = c(Nat.0) * d(Nat.0 - Nat.0)
    Nat.0 - Nat.0 = Nat.0
    coeff_convolution(c, d, Nat.0) = c(Nat.0) * d(Nat.0)
}

lemma eval_mul_algebra_step[R: CommRing](a: R, b: R, u: R, v: R, w: R, x: R) {
    w = b + x * u implies a * b + x * (a * u + v * w) = (a + x * v) * w
} by {
    if w = b + x * u {
        x * (a * u + v * w) = x * (a * u) + x * (v * w)
        a * b + x * (a * u + v * w) = a * b + (x * (a * u) + x * (v * w))
        a * b + (x * (a * u) + x * (v * w)) = a * b + x * (a * u) + x * (v * w)
        Semigroup.mul_associative[R](a, x, u)
        a * (x * u) = a * x * u
        CommSemigroup.commutative[R](a, x)
        a * x = x * a
        a * x * u = x * a * u
        Semigroup.mul_associative[R](x, a, u)
        x * (a * u) = x * a * u
        a * (x * u) = x * (a * u)
        a * w = a * (b + x * u)
        a * (b + x * u) = a * b + a * (x * u)
        a * w = a * b + x * (a * u)
        Semigroup.mul_associative[R](x, v, w)
        x * (v * w) = x * v * w
        (x * v) * w = x * v * w
        (x * v) * w = x * (v * w)
        (a + x * v) * w = a * w + (x * v) * w
        (a + x * v) * w = a * b + x * (a * u) + x * (v * w)
        a * b + x * (a * u + v * w) = (a + x * v) * w
    }
}

define coeff_eval_convolution_mul_at[R: CommRing](c: Nat -> R, d: Nat -> R, x: R, m: Nat, n: Nat) -> Bool {
    coeff_zero_from(c, m) and coeff_zero_from(d, n) implies
    coeff_eval(coeff_convolution(c, d), x, m + n) = coeff_eval(c, x, m) * coeff_eval(d, x, n)
}

define coeff_eval_convolution_mul_property[R: CommRing](x: R, m: Nat) -> Bool {
    forall(c: Nat -> R, d: Nat -> R, n: Nat) {
        coeff_eval_convolution_mul_at(c, d, x, m, n)
    }
}

lemma coeff_eval_convolution_mul_property_apply[R: CommRing](x: R, m: Nat, c: Nat -> R, d: Nat -> R, n: Nat) {
    coeff_eval_convolution_mul_property(x, m) implies coeff_eval_convolution_mul_at(c, d, x, m, n)
} by {
    if coeff_eval_convolution_mul_property(x, m) {
        coeff_eval_convolution_mul_property(x, m) = forall(c0: Nat -> R, d0: Nat -> R, n0: Nat) {
            coeff_eval_convolution_mul_at(c0, d0, x, m, n0)
        }
        let c_arg = c
        let d_arg = d
        let n_arg = n
        c_arg = c
        d_arg = d
        n_arg = n
        coeff_eval_convolution_mul_at(c_arg, d_arg, x, m, n_arg)
        coeff_eval_convolution_mul_at(c, d, x, m, n)
    }
}

lemma coeff_eval_convolution_mul_property_intro[R: CommRing](x: R, m: Nat) {
    (forall(c: Nat -> R, d: Nat -> R, n: Nat) {
        coeff_eval_convolution_mul_at(c, d, x, m, n)
    }) implies coeff_eval_convolution_mul_property(x, m)
} by {
    forall(c: Nat -> R, d: Nat -> R, n: Nat) {
        coeff_eval_convolution_mul_at(c, d, x, m, n)
    }
    coeff_eval_convolution_mul_property(x, m) = forall(c: Nat -> R, d: Nat -> R, n: Nat) {
        coeff_eval_convolution_mul_at(c, d, x, m, n)
    }
}

lemma coeff_eval_convolution_mul_of_zero_from[R: CommRing](c: Nat -> R, d: Nat -> R, x: R, m: Nat, n: Nat) {
    coeff_zero_from(c, m) and coeff_zero_from(d, n) implies
    coeff_eval(coeff_convolution(c, d), x, m + n) = coeff_eval(c, x, m) * coeff_eval(d, x, n)
} by {
    define statement(bound: Nat) -> Bool {
        coeff_eval_convolution_mul_property(x, bound)
    }

    forall(c0: Nat -> R, d0: Nat -> R, n0: Nat) {
        if coeff_zero_from(c0, Nat.0) and coeff_zero_from(d0, n0) {
            eval_mul_coeff_convolution_zero_from_left_zero_from_zero(c0, d0)
            coeff_zero_from(coeff_convolution(c0, d0), Nat.0)
            Nat.0 <= n0
            Nat.0 + n0 = n0
            coeff_eval_eq_of_zero_from_le(coeff_convolution(c0, d0), x, Nat.0, n0)
            coeff_eval(coeff_convolution(c0, d0), x, n0) = coeff_eval(coeff_convolution(c0, d0), x, Nat.0)
            coeff_eval(coeff_convolution(c0, d0), x, Nat.0) = R.0
            coeff_eval(c0, x, Nat.0) = R.0
            R.0 * coeff_eval(d0, x, n0) = R.0
            coeff_eval(c0, x, Nat.0) * coeff_eval(d0, x, n0) = R.0
            coeff_eval(coeff_convolution(c0, d0), x, Nat.0 + n0) =
                coeff_eval(c0, x, Nat.0) * coeff_eval(d0, x, n0)
        }
        coeff_eval_convolution_mul_at(c0, d0, x, Nat.0, n0) =
            (coeff_zero_from(c0, Nat.0) and coeff_zero_from(d0, n0) implies
            coeff_eval(coeff_convolution(c0, d0), x, Nat.0 + n0) =
                coeff_eval(c0, x, Nat.0) * coeff_eval(d0, x, n0))
        coeff_eval_convolution_mul_at(c0, d0, x, Nat.0, n0)
    }
    coeff_eval_convolution_mul_property_intro(x, Nat.0)
    coeff_eval_convolution_mul_property(x, Nat.0)
    statement(Nat.0)

    forall(bound: Nat) {
        if statement(bound) {
            forall(c0: Nat -> R, d0: Nat -> R, n0: Nat) {
                if coeff_zero_from(c0, bound.suc) and coeff_zero_from(d0, n0) {
                    coeff_zero_from(c0, bound.suc) and coeff_zero_from(d0, n0)
                    let ct = coeff_tail(c0)
                    let dt = coeff_tail(d0)
                    let conv = coeff_convolution(c0, d0)
                    let conv_tail = coeff_convolution(ct, d0)
                    let scalar_tail = eval_mul_scalar_coeff(c0(Nat.0), dt)
                    ct = coeff_tail(c0)
                    dt = coeff_tail(d0)
                    conv = coeff_convolution(c0, d0)
                    conv_tail = coeff_convolution(ct, d0)
                    scalar_tail = eval_mul_scalar_coeff(c0(Nat.0), dt)

                    add_suc_left(bound, n0)
                    bound.suc + n0 = (bound + n0).suc
                    coeff_eval(conv, x, bound.suc + n0) = coeff_eval(conv, x, (bound + n0).suc)
                    coeff_eval(conv, x, (bound + n0).suc) =
                        conv(Nat.0) + x * coeff_eval(coeff_tail(conv), x, bound + n0)
                    eval_mul_convolution_zero(c0, d0)
                    conv(Nat.0) = c0(Nat.0) * d0(Nat.0)
                    eval_mul_coeff_convolution_tail(c0, d0)
                    coeff_tail(conv) = pointwise_add(scalar_tail, conv_tail)
                    coeff_eval(coeff_tail(conv), x, bound + n0) =
                        coeff_eval(pointwise_add(scalar_tail, conv_tail), x, bound + n0)
                    coeff_eval_add(scalar_tail, conv_tail, x, bound + n0)
                    coeff_eval(pointwise_add(scalar_tail, conv_tail), x, bound + n0) =
                        coeff_eval(scalar_tail, x, bound + n0) + coeff_eval(conv_tail, x, bound + n0)

                    eval_mul_coeff_zero_from_tail_pred(c0, bound)
                    coeff_zero_from(ct, bound)
                    statement(bound) = coeff_eval_convolution_mul_property(x, bound)
                    coeff_eval_convolution_mul_property_apply(x, bound, ct, d0, n0)
                    coeff_eval_convolution_mul_at(ct, d0, x, bound, n0)
                    coeff_eval_convolution_mul_at(ct, d0, x, bound, n0) =
                        (coeff_zero_from(ct, bound) and coeff_zero_from(d0, n0) implies
                        coeff_eval(coeff_convolution(ct, d0), x, bound + n0) =
                            coeff_eval(ct, x, bound) * coeff_eval(d0, x, n0))
                    coeff_eval(coeff_convolution(ct, d0), x, bound + n0) =
                        coeff_eval(ct, x, bound) * coeff_eval(d0, x, n0)
                    conv_tail = coeff_convolution(ct, d0)
                    coeff_eval(conv_tail, x, bound + n0) = coeff_eval(ct, x, bound) * coeff_eval(d0, x, n0)

                    eval_mul_coeff_zero_from_tail_same_bound(d0, n0)
                    coeff_zero_from(dt, n0)
                    n0 + bound = bound + n0
                    n0 <= bound + n0
                    eval_mul_scalar_coeff_eval_of_zero_from_le(c0(Nat.0), dt, x, n0, bound + n0)
                    coeff_eval(scalar_tail, x, bound + n0) = c0(Nat.0) * coeff_eval(dt, x, n0)
                    eval_mul_coeff_eval_head_tail_same_bound(d0, x, n0)
                    coeff_eval(d0, x, n0) = d0(Nat.0) + x * coeff_eval(dt, x, n0)

                    coeff_eval(conv, x, bound.suc + n0) =
                        c0(Nat.0) * d0(Nat.0) +
                        x * (c0(Nat.0) * coeff_eval(dt, x, n0) +
                            coeff_eval(ct, x, bound) * coeff_eval(d0, x, n0))
                    eval_mul_algebra_step(
                        c0(Nat.0),
                        d0(Nat.0),
                        coeff_eval(dt, x, n0),
                        coeff_eval(ct, x, bound),
                        coeff_eval(d0, x, n0),
                        x
                    )
                    c0(Nat.0) * d0(Nat.0) +
                        x * (c0(Nat.0) * coeff_eval(dt, x, n0) +
                            coeff_eval(ct, x, bound) * coeff_eval(d0, x, n0)) =
                        (c0(Nat.0) + x * coeff_eval(ct, x, bound)) * coeff_eval(d0, x, n0)
                    coeff_eval(c0, x, bound.suc) = c0(Nat.0) + x * coeff_eval(ct, x, bound)
                    coeff_eval(conv, x, bound.suc + n0) = coeff_eval(c0, x, bound.suc) * coeff_eval(d0, x, n0)
                    coeff_eval(coeff_convolution(c0, d0), x, bound.suc + n0) =
                        coeff_eval(c0, x, bound.suc) * coeff_eval(d0, x, n0)
                }
                coeff_eval_convolution_mul_at(c0, d0, x, bound.suc, n0) =
                    (coeff_zero_from(c0, bound.suc) and coeff_zero_from(d0, n0) implies
                    coeff_eval(coeff_convolution(c0, d0), x, bound.suc + n0) =
                        coeff_eval(c0, x, bound.suc) * coeff_eval(d0, x, n0))
                coeff_eval_convolution_mul_at(c0, d0, x, bound.suc, n0)
            }
            coeff_eval_convolution_mul_property_intro(x, bound.suc)
            coeff_eval_convolution_mul_property(x, bound.suc)
            statement(bound.suc)
        }
    }

    statement(Nat.0) and forall(k: Nat) { statement(k) implies statement(k.suc) }
    alt_induction(statement)
    forall(k: Nat) { statement(k) }
    statement(m)
    coeff_eval_convolution_mul_property_apply(x, m, c, d, n)
    coeff_eval_convolution_mul_at(c, d, x, m, n)
    coeff_eval_convolution_mul_at(c, d, x, m, n) =
        (coeff_zero_from(c, m) and coeff_zero_from(d, n) implies
        coeff_eval(coeff_convolution(c, d), x, m + n) = coeff_eval(c, x, m) * coeff_eval(d, x, n))
}

lemma polynomial_mul_coeff_eq_coeff_convolution[R: CommRing](p: Polynomial[R], q: Polynomial[R]) {
    polynomial_mul_coeff(p, q) = coeff_convolution(p.coeff, q.coeff)
} by {
    forall(k: Nat) {
        let left_term: Nat -> R = function(i: Nat) { lib(polynomial.mul).polynomial_mul_term_coeff(p, q, k, i) }
        let right_term: Nat -> R = function(i: Nat) { coeff_convolution_term(p.coeff, q.coeff, k, i) }
        forall(i: Nat) {
            if i < k.suc {
                left_term(i) = p.coeff(i) * q.coeff(k - i)
                right_term(i) = p.coeff(i) * q.coeff(k - i)
                left_term(i) = right_term(i)
            }
        }
        partial_pointwise_eq[R](left_term, right_term, k.suc)
        partial(left_term, k.suc) = partial(right_term, k.suc)
        polynomial_mul_coeff(p, q, k) = partial(left_term, k.suc)
        coeff_convolution(p.coeff, q.coeff, k) = partial(right_term, k.suc)
        polynomial_mul_coeff(p, q, k) = coeff_convolution(p.coeff, q.coeff, k)
    }
    function_extensionality(polynomial_mul_coeff(p, q), coeff_convolution(p.coeff, q.coeff))
}

/// Bounded evaluation of a polynomial product is the product of bounded evaluations.
theorem polynomial_eval_bound_mul_of_support_bounded[R: CommRing](
    p: Polynomial[R],
    q: Polynomial[R],
    x: R,
    m: Nat,
    n: Nat
) {
    polynomial_support_bounded_by(p, m) and polynomial_support_bounded_by(q, n) implies
    polynomial_eval_bound(polynomial_mul(p, q), x, m + n) =
        polynomial_eval_bound(p, x, m) * polynomial_eval_bound(q, x, n)
} by {
    if polynomial_support_bounded_by(p, m) and polynomial_support_bounded_by(q, n) {
        polynomial_eval_bound_eq_coeff_eval(polynomial_mul(p, q), x, m + n)
        polynomial_eval_bound(polynomial_mul(p, q), x, m + n) = coeff_eval(polynomial_mul(p, q).coeff, x, m + n)
        polynomial_mul_coeff_eq(p, q)
        polynomial_mul(p, q).coeff = polynomial_mul_coeff(p, q)
        polynomial_mul_coeff_eq_coeff_convolution(p, q)
        polynomial_mul(p, q).coeff = coeff_convolution(p.coeff, q.coeff)
        polynomial_support_bounded_by(p, m) = coeff_zero_from(p.coeff, m)
        polynomial_support_bounded_by(q, n) = coeff_zero_from(q.coeff, n)
        coeff_eval_convolution_mul_of_zero_from(p.coeff, q.coeff, x, m, n)
        coeff_eval(coeff_convolution(p.coeff, q.coeff), x, m + n) =
            coeff_eval(p.coeff, x, m) * coeff_eval(q.coeff, x, n)
        polynomial_eval_bound_eq_coeff_eval(p, x, m)
        polynomial_eval_bound_eq_coeff_eval(q, x, n)
        polynomial_eval_bound(p, x, m) = coeff_eval(p.coeff, x, m)
        polynomial_eval_bound(q, x, n) = coeff_eval(q.coeff, x, n)
        polynomial_eval_bound(polynomial_mul(p, q), x, m + n) =
            polynomial_eval_bound(p, x, m) * polynomial_eval_bound(q, x, n)
    }
}

/// Global evaluation of a polynomial product is the product of global evaluations.
theorem polynomial_eval_mul[R: CommRing](p: Polynomial[R], q: Polynomial[R], x: R) {
    polynomial_eval(polynomial_mul(p, q), x) = polynomial_eval(p, x) * polynomial_eval(q, x)
} by {
    polynomial_support_bound_exists(p)
    polynomial_support_bound_exists(q)
    let m: Nat satisfy {
        polynomial_support_bounded_by(p, m)
    }
    let n: Nat satisfy {
        polynomial_support_bounded_by(q, n)
    }
    polynomial_mul_support_bounded_by_add(p, q, m, n)
    polynomial_support_bounded_by(polynomial_mul(p, q), m + n)
    polynomial_eval_eq_eval_bound_of_support_bounded(polynomial_mul(p, q), x, m + n)
    polynomial_eval_eq_eval_bound_of_support_bounded(p, x, m)
    polynomial_eval_eq_eval_bound_of_support_bounded(q, x, n)
    polynomial_eval_bound_mul_of_support_bounded(p, q, x, m, n)
    polynomial_eval_bound(polynomial_mul(p, q), x, m + n) =
        polynomial_eval_bound(p, x, m) * polynomial_eval_bound(q, x, n)
    polynomial_eval(polynomial_mul(p, q), x) = polynomial_eval_bound(polynomial_mul(p, q), x, m + n)
    polynomial_eval(p, x) = polynomial_eval_bound(p, x, m)
    polynomial_eval(q, x) = polynomial_eval_bound(q, x, n)
    polynomial_eval(polynomial_mul(p, q), x) = polynomial_eval(p, x) * polynomial_eval(q, x)
}
