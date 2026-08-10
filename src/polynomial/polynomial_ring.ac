/// Polynomial rings: the degree bounds for sums and products, evaluation as
/// a ring homomorphism, the remainder theorem for quadratics, and the
/// integral domain property over a field.
///
/// The library has no separate degree function: a polynomial is "of degree
/// at most `d`" exactly when its support lies below `d + 1`
/// (`polynomial_support_bounded_by`), which is the convention used
/// throughout this file.

from nat import Nat, add_sub, add_cancels_right, lt_suc,
    lt_suc_right, lt_or_lte, lt_and_lte, lte_suc_suc,
    lt_trans, lt_not_symm, lt_not_ref, lte_imp_not_lt, lte_trans,
    zero_or_suc, trichotomy, not_lt_zero
from order import lt_imp_lte, lte_max_left, lte_max_right
from semiring import Semiring
from comm_ring import CommRing
from algebra.field.field import Field, field_mul_eq_zero, field_mul_nonzero
from algebra.ring.ring import mul_neg_left
from algebra.add_group import inverse_left, inverse_add
from polynomial.base import Polynomial, polynomial_constant, polynomial_monomial,
    polynomial_add_coeff, polynomial_eq_coeff_at, polynomial_constant_coeff_zero,
    polynomial_constant_coeff_of_ne_zero, polynomial_monomial_coeff_self,
    polynomial_monomial_coeff_of_ne,
    polynomial_ext_pointwise, polynomial_eq_zero_of_coeff_zero, polynomial_zero_coeff
from polynomial.coeff_eval import coeff_eval, coeff_tail
from polynomial.add_monoid_algebra_bridge import coeff_zero_at, coeff_zero_from,
    coeff_zero_from_at, coeff_zero_from_apply
from polynomial.eval import polynomial_support_bounded_by, polynomial_add_support_bounded_by,
    polynomial_support_bounded_by_monotone, polynomial_support_bounded_by_apply,
    polynomial_constant_support_bounded_by_one,
    polynomial_eval_bound, polynomial_eval_bound_eq_coeff_eval
from polynomial.global_eval import polynomial_eval, polynomial_eval_add, polynomial_eval_constant,
    polynomial_eval_eq_eval_bound_of_support_bounded
from polynomial.eval_mul import polynomial_eval_mul
from polynomial.mul import polynomial_mul, polynomial_mul_coeff, polynomial_mul_term_coeff,
    polynomial_mul_coeff_apply, polynomial_mul_support_bounded_by_add,
    polynomial_mul_coeff_zero_of_support_bounded
from polynomial.global_factor import polynomial_divisible_by_x_minus_a_polynomial,
    polynomial_x_minus_a_polynomial_witness, polynomial_x_minus_a_polynomial_witness_at,
    polynomial_factor_theorem_forward_global,
    polynomial_factor_theorem_reverse_global, polynomial_factor_theorem_global
from list import partial
from data.basic.functions import function_extensionality
from data.nat.nat_range_sum import range_sum, range_sum_one, range_sum_suc
from data.nat.nat_range_sum_bridge import range_sum_eq_partial

numerals Nat

/// A concrete subtraction fact: `c + b = a` witnesses `a - b = c`.
lemma ring_nat_sub_eq_of_add(a: Nat, b: Nat, c: Nat) {
    c + b = a implies a - b = c
} by {
    if c + b = a {
        b + c = c + b
        b + c = a
        exists(c0: Nat) {
            c0 = c and b + c0 = a
        }
        b <= a
        add_sub(a, b)
        a - b + b = a
        a - b + b = c + b
        b + (a - b) = a - b + b
        b + (a - b) = c + b
        add_cancels_right(a - b, b, c)
        a - b = c
    }
}

/// A zero coefficient outside a bound witnesses the bounded-coefficient
/// assertion at that index.
lemma ring_coeff_zero_from_at_intro[R: Semiring](c: Nat -> R, n: Nat, k: Nat) {
    not k < n and c(k) = R.0 implies coeff_zero_from_at(c, n, k)
} by {
    if not k < n and c(k) = R.0 {
        coeff_zero_at(c, k)
        coeff_zero_from_at(c, n, k) = (not k < n implies coeff_zero_at(c, k))
        coeff_zero_from_at(c, n, k)
    }
}

/// A pointwise bounded-coefficient assertion gives the bounded-coefficient
/// predicate.
lemma ring_coeff_zero_from_intro[R: Semiring](c: Nat -> R, n: Nat) {
    (forall(k: Nat) { coeff_zero_from_at(c, n, k) }) implies coeff_zero_from(c, n)
} by {
    forall(k: Nat) {
        coeff_zero_from_at(c, n, k)
    }
    coeff_zero_from(c, n) = forall(k: Nat) {
        coeff_zero_from_at(c, n, k)
    }
}

/// The linear polynomial `a * X + b`.
define linear_polynomial[R: Semiring](a: R, b: R) -> Polynomial[R] {
    polynomial_constant(b) + polynomial_monomial(Nat.1, a)
}

/// The quadratic polynomial `a * X^2 + b * X + c`.
define quadratic_polynomial[R: Semiring](a: R, b: R, c: R) -> Polynomial[R] {
    polynomial_constant(c) + polynomial_monomial(Nat.1, b) + polynomial_monomial(Nat.2, a)
}

// ---------------------------------------------------------------------------
// Degrees of sums and products
// ---------------------------------------------------------------------------

/// The degree of a polynomial is at most `d` when the support lies below `d + 1`.
define polynomial_degree_le[R: Semiring](p: Polynomial[R], d: Nat) -> Bool {
    polynomial_support_bounded_by(p, d.suc)
}

/// A monomial is support-bounded by one more than its exponent.
theorem polynomial_monomial_support_bounded_by_suc[R: Semiring](n: Nat, r: R) {
    polynomial_support_bounded_by(Polynomial[R].monomial(n, r), n.suc)
} by {
    forall(k: Nat) {
        if k < n.suc {
            coeff_zero_from_at(Polynomial[R].monomial(n, r).coeff, n.suc, k) =
                (not k < n.suc implies coeff_zero_at(Polynomial[R].monomial(n, r).coeff, k))
            coeff_zero_from_at(Polynomial[R].monomial(n, r).coeff, n.suc, k)
        } else {
            not k < n.suc
            lt_or_lte(k, n.suc)
            n.suc <= k
            if k = n {
                lt_suc(n)
                n < n.suc
                lt_and_lte(n, n.suc, k)
                n < k
                lt_not_symm(n, k)
                not k < n
                lt_and_lte(k, n, k)
                k < k
                lt_not_symm(k, k)
                not k < k
                false
            }
            k != n
            polynomial_monomial_coeff_of_ne[R](n, r, k)
            Polynomial[R].monomial(n, r).coeff(k) = R.0
            coeff_zero_at(Polynomial[R].monomial(n, r).coeff, k)
            coeff_zero_from_at(Polynomial[R].monomial(n, r).coeff, n.suc, k) =
                (not k < n.suc implies coeff_zero_at(Polynomial[R].monomial(n, r).coeff, k))
            coeff_zero_from_at(Polynomial[R].monomial(n, r).coeff, n.suc, k)
        }
    }
    coeff_zero_from(Polynomial[R].monomial(n, r).coeff, n.suc) = forall(k: Nat) {
        coeff_zero_from_at(Polynomial[R].monomial(n, r).coeff, n.suc, k)
    }
    coeff_zero_from(Polynomial[R].monomial(n, r).coeff, n.suc)
    polynomial_support_bounded_by(Polynomial[R].monomial(n, r), n.suc) =
        coeff_zero_from(Polynomial[R].monomial(n, r).coeff, n.suc)
    polynomial_support_bounded_by(Polynomial[R].monomial(n, r), n.suc)
}

/// Two is at most three.
lemma ring_nat_two_le_three {
    Nat.2 <= Nat.3
} by {
    lt_suc(Nat.2)
    Nat.2 < Nat.2.suc
    Nat.2.suc = Nat.3
    lt_imp_lte[Nat](Nat.2, Nat.3)
    Nat.2 <= Nat.3
}

/// One is at most three.
lemma ring_nat_one_le_three {
    Nat.1 <= Nat.3
} by {
    lt_suc(Nat.1)
    Nat.1 < Nat.1.suc
    Nat.1.suc = Nat.2
    lt_imp_lte[Nat](Nat.1, Nat.2)
    Nat.1 <= Nat.2
    lt_suc(Nat.2)
    Nat.2 < Nat.2.suc
    Nat.2.suc = Nat.3
    lt_imp_lte[Nat](Nat.2, Nat.3)
    Nat.2 <= Nat.3
    lte_trans(Nat.1, Nat.2, Nat.3)
    Nat.1 <= Nat.3
}

/// The constant coefficient of a linear polynomial is its constant term.
theorem linear_polynomial_coeff_zero[R: Semiring](a: R, b: R) {
    linear_polynomial(a, b).coeff(Nat.0) = b
} by {
    polynomial_add_coeff(polynomial_constant(b), polynomial_monomial(Nat.1, a), Nat.0)
    linear_polynomial(a, b).coeff(Nat.0) =
        polynomial_constant(b).coeff(Nat.0) + polynomial_monomial(Nat.1, a).coeff(Nat.0)
    polynomial_constant_coeff_zero(b)
    polynomial_constant(b).coeff(Nat.0) = b
    Nat.0 != Nat.1
    polynomial_monomial_coeff_of_ne(Nat.1, a, Nat.0)
    polynomial_monomial(Nat.1, a).coeff(Nat.0) = R.0
    linear_polynomial(a, b).coeff(Nat.0) = b + R.0
    b + R.0 = b
    linear_polynomial(a, b).coeff(Nat.0) = b
}

/// The linear coefficient of a linear polynomial is its leading term.
theorem linear_polynomial_coeff_one[R: Semiring](a: R, b: R) {
    linear_polynomial(a, b).coeff(Nat.1) = a
} by {
    polynomial_add_coeff(polynomial_constant(b), polynomial_monomial(Nat.1, a), Nat.1)
    linear_polynomial(a, b).coeff(Nat.1) =
        polynomial_constant(b).coeff(Nat.1) + polynomial_monomial(Nat.1, a).coeff(Nat.1)
    Nat.1 != Nat.0
    polynomial_constant_coeff_of_ne_zero(b, Nat.1)
    polynomial_constant(b).coeff(Nat.1) = R.0
    polynomial_monomial_coeff_self(Nat.1, a)
    polynomial_monomial(Nat.1, a).coeff(Nat.1) = a
    linear_polynomial(a, b).coeff(Nat.1) = R.0 + a
    R.0 + a = a
    linear_polynomial(a, b).coeff(Nat.1) = a
}

/// A linear polynomial is supported below two coefficient slots.
theorem linear_polynomial_support_bounded_by_two[R: Semiring](a: R, b: R) {
    polynomial_support_bounded_by(linear_polynomial(a, b), Nat.2)
} by {
    polynomial_constant_support_bounded_by_one(b)
    polynomial_support_bounded_by(polynomial_constant(b), Nat.1)
    polynomial_monomial_support_bounded_by_suc(Nat.1, a)
    polynomial_support_bounded_by(polynomial_monomial(Nat.1, a), Nat.2)
    lt_suc(Nat.1)
    Nat.1 < Nat.1.suc
    Nat.1.suc = Nat.2
    lt_imp_lte[Nat](Nat.1, Nat.2)
    Nat.1 <= Nat.2
    polynomial_support_bounded_by_monotone(polynomial_constant(b), Nat.1, Nat.2)
    polynomial_support_bounded_by(polynomial_constant(b), Nat.2)
    polynomial_add_support_bounded_by(polynomial_constant(b), polynomial_monomial(Nat.1, a), Nat.2)
    polynomial_support_bounded_by(polynomial_constant(b) + polynomial_monomial(Nat.1, a), Nat.2)
    polynomial_support_bounded_by(linear_polynomial(a, b), Nat.2)
}

/// The constant coefficient of a quadratic is its constant term.
theorem quadratic_polynomial_coeff_zero[R: Semiring](a: R, b: R, c: R) {
    quadratic_polynomial(a, b, c).coeff(Nat.0) = c
} by {
    polynomial_add_coeff(polynomial_constant(c) + polynomial_monomial(Nat.1, b),
        polynomial_monomial(Nat.2, a), Nat.0)
    quadratic_polynomial(a, b, c).coeff(Nat.0) =
        (polynomial_constant(c) + polynomial_monomial(Nat.1, b)).coeff(Nat.0) +
        polynomial_monomial(Nat.2, a).coeff(Nat.0)
    polynomial_add_coeff(polynomial_constant(c), polynomial_monomial(Nat.1, b), Nat.0)
    (polynomial_constant(c) + polynomial_monomial(Nat.1, b)).coeff(Nat.0) =
        polynomial_constant(c).coeff(Nat.0) + polynomial_monomial(Nat.1, b).coeff(Nat.0)
    polynomial_constant_coeff_zero(c)
    polynomial_constant(c).coeff(Nat.0) = c
    Nat.0 != Nat.1
    polynomial_monomial_coeff_of_ne(Nat.1, b, Nat.0)
    polynomial_monomial(Nat.1, b).coeff(Nat.0) = R.0
    Nat.0 != Nat.2
    polynomial_monomial_coeff_of_ne(Nat.2, a, Nat.0)
    polynomial_monomial(Nat.2, a).coeff(Nat.0) = R.0
    quadratic_polynomial(a, b, c).coeff(Nat.0) = c + R.0 + R.0
    c + R.0 + R.0 = c
    quadratic_polynomial(a, b, c).coeff(Nat.0) = c
}

/// The linear coefficient of a quadratic is its middle term.
theorem quadratic_polynomial_coeff_one[R: Semiring](a: R, b: R, c: R) {
    quadratic_polynomial(a, b, c).coeff(Nat.1) = b
} by {
    polynomial_add_coeff(polynomial_constant(c) + polynomial_monomial(Nat.1, b),
        polynomial_monomial(Nat.2, a), Nat.1)
    quadratic_polynomial(a, b, c).coeff(Nat.1) =
        (polynomial_constant(c) + polynomial_monomial(Nat.1, b)).coeff(Nat.1) +
        polynomial_monomial(Nat.2, a).coeff(Nat.1)
    polynomial_add_coeff(polynomial_constant(c), polynomial_monomial(Nat.1, b), Nat.1)
    (polynomial_constant(c) + polynomial_monomial(Nat.1, b)).coeff(Nat.1) =
        polynomial_constant(c).coeff(Nat.1) + polynomial_monomial(Nat.1, b).coeff(Nat.1)
    Nat.1 != Nat.0
    polynomial_constant_coeff_of_ne_zero(c, Nat.1)
    polynomial_constant(c).coeff(Nat.1) = R.0
    polynomial_monomial_coeff_self(Nat.1, b)
    polynomial_monomial(Nat.1, b).coeff(Nat.1) = b
    Nat.1 != Nat.2
    polynomial_monomial_coeff_of_ne(Nat.2, a, Nat.1)
    polynomial_monomial(Nat.2, a).coeff(Nat.1) = R.0
    quadratic_polynomial(a, b, c).coeff(Nat.1) = R.0 + b + R.0
    R.0 + b + R.0 = b
    quadratic_polynomial(a, b, c).coeff(Nat.1) = b
}

/// The quadratic coefficient of a quadratic is its leading term.
theorem quadratic_polynomial_coeff_two[R: Semiring](a: R, b: R, c: R) {
    quadratic_polynomial(a, b, c).coeff(Nat.2) = a
} by {
    polynomial_add_coeff(polynomial_constant(c) + polynomial_monomial(Nat.1, b),
        polynomial_monomial(Nat.2, a), Nat.2)
    quadratic_polynomial(a, b, c).coeff(Nat.2) =
        (polynomial_constant(c) + polynomial_monomial(Nat.1, b)).coeff(Nat.2) +
        polynomial_monomial(Nat.2, a).coeff(Nat.2)
    polynomial_add_coeff(polynomial_constant(c), polynomial_monomial(Nat.1, b), Nat.2)
    (polynomial_constant(c) + polynomial_monomial(Nat.1, b)).coeff(Nat.2) =
        polynomial_constant(c).coeff(Nat.2) + polynomial_monomial(Nat.1, b).coeff(Nat.2)
    Nat.2 != Nat.0
    polynomial_constant_coeff_of_ne_zero(c, Nat.2)
    polynomial_constant(c).coeff(Nat.2) = R.0
    Nat.1 != Nat.2
    polynomial_monomial_coeff_of_ne(Nat.1, b, Nat.2)
    polynomial_monomial(Nat.1, b).coeff(Nat.2) = R.0
    polynomial_monomial_coeff_self(Nat.2, a)
    polynomial_monomial(Nat.2, a).coeff(Nat.2) = a
    quadratic_polynomial(a, b, c).coeff(Nat.2) = R.0 + R.0 + a
    R.0 + R.0 + a = a
    quadratic_polynomial(a, b, c).coeff(Nat.2) = a
}

/// A quadratic polynomial is supported below three coefficient slots.
theorem quadratic_polynomial_support_bounded_by_three[R: Semiring](a: R, b: R, c: R) {
    polynomial_support_bounded_by(quadratic_polynomial(a, b, c), Nat.3)
} by {
    polynomial_constant_support_bounded_by_one(c)
    polynomial_support_bounded_by(polynomial_constant(c), Nat.1)
    ring_nat_one_le_three
    polynomial_support_bounded_by_monotone(polynomial_constant(c), Nat.1, Nat.3)
    polynomial_support_bounded_by(polynomial_constant(c), Nat.3)
    polynomial_monomial_support_bounded_by_suc(Nat.1, b)
    polynomial_support_bounded_by(polynomial_monomial(Nat.1, b), Nat.2)
    ring_nat_two_le_three
    polynomial_support_bounded_by_monotone(polynomial_monomial(Nat.1, b), Nat.2, Nat.3)
    polynomial_support_bounded_by(polynomial_monomial(Nat.1, b), Nat.3)
    polynomial_add_support_bounded_by(polynomial_constant(c), polynomial_monomial(Nat.1, b), Nat.3)
    polynomial_support_bounded_by(polynomial_constant(c) + polynomial_monomial(Nat.1, b), Nat.3)
    polynomial_monomial_support_bounded_by_suc(Nat.2, a)
    polynomial_support_bounded_by(polynomial_monomial(Nat.2, a), Nat.3)
    polynomial_add_support_bounded_by(polynomial_constant(c) + polynomial_monomial(Nat.1, b),
        polynomial_monomial(Nat.2, a), Nat.3)
    polynomial_support_bounded_by(
        (polynomial_constant(c) + polynomial_monomial(Nat.1, b)) + polynomial_monomial(Nat.2, a),
        Nat.3)
    polynomial_support_bounded_by(quadratic_polynomial(a, b, c), Nat.3)
}

/// The classical bound for the degree of a sum:
/// `deg(f + g) <= max(deg f, deg g)`.
theorem polynomial_degree_sum_le_max[R: Semiring](
    p: Polynomial[R], q: Polynomial[R], m: Nat, n: Nat
) {
    polynomial_degree_le(p, m) and polynomial_degree_le(q, n) implies
    polynomial_degree_le(p + q, m.max(n))
} by {
    if polynomial_degree_le(p, m) and polynomial_degree_le(q, n) {
        polynomial_degree_le(p, m) = polynomial_support_bounded_by(p, m.suc)
        polynomial_support_bounded_by(p, m.suc)
        lte_max_left(m, n)
        m <= m.max(n)
        lte_suc_suc(m, m.max(n))
        m.suc <= m.max(n).suc
        polynomial_support_bounded_by_monotone(p, m.suc, m.max(n).suc)
        polynomial_support_bounded_by(p, m.max(n).suc)
        polynomial_degree_le(q, n) = polynomial_support_bounded_by(q, n.suc)
        polynomial_support_bounded_by(q, n.suc)
        lte_max_right(m, n)
        n <= m.max(n)
        lte_suc_suc(n, m.max(n))
        n.suc <= m.max(n).suc
        polynomial_support_bounded_by_monotone(q, n.suc, m.max(n).suc)
        polynomial_support_bounded_by(q, m.max(n).suc)
        polynomial_add_support_bounded_by(p, q, m.max(n).suc)
        polynomial_support_bounded_by(p + q, m.max(n).suc)
        polynomial_degree_le(p + q, m.max(n)) =
            polynomial_support_bounded_by(p + q, m.max(n).suc)
        polynomial_degree_le(p + q, m.max(n))
    }
}

// ---------------------------------------------------------------------------
// Evaluation of monomials, linear and quadratic polynomials
// ---------------------------------------------------------------------------

/// Bounded evaluation at bound zero is the zero of the coefficient ring.
lemma ring_coeff_eval_nat_zero[R: Semiring](c: Nat -> R, x: R) {
    coeff_eval(c, x, Nat.0) = R.0
}

/// Evaluation of a monomial of degree one at a point is scalar multiplication.
theorem monomial_one_eval[R: CommRing](r: R, x: R) {
    polynomial_eval(Polynomial[R].monomial(Nat.1, r), x) = r * x
} by {
    polynomial_monomial_support_bounded_by_suc(Nat.1, r)
    polynomial_support_bounded_by(Polynomial[R].monomial(Nat.1, r), Nat.0.suc.suc)
    polynomial_eval_eq_eval_bound_of_support_bounded(
        Polynomial[R].monomial(Nat.1, r), x, Nat.0.suc.suc)
    polynomial_eval(Polynomial[R].monomial(Nat.1, r), x) =
        polynomial_eval_bound(Polynomial[R].monomial(Nat.1, r), x, Nat.0.suc.suc)
    polynomial_eval_bound_eq_coeff_eval(Polynomial[R].monomial(Nat.1, r), x, Nat.0.suc.suc)
    coeff_eval(Polynomial[R].monomial(Nat.1, r).coeff, x, Nat.0.suc.suc) =
        Polynomial[R].monomial(Nat.1, r).coeff(Nat.0) +
        x * coeff_eval(coeff_tail(Polynomial[R].monomial(Nat.1, r).coeff), x, Nat.0.suc)
    polynomial_monomial_coeff_of_ne(Nat.1, r, Nat.0)
    Nat.0 != Nat.1
    Polynomial[R].monomial(Nat.1, r).coeff(Nat.0) = R.0
    coeff_eval(coeff_tail(Polynomial[R].monomial(Nat.1, r).coeff), x, Nat.0.suc) =
        Polynomial[R].monomial(Nat.1, r).coeff(Nat.1) +
        x * coeff_eval(coeff_tail(coeff_tail(Polynomial[R].monomial(Nat.1, r).coeff)), x, Nat.0)
    Polynomial[R].monomial(Nat.1, r).coeff(Nat.1) = r
    ring_coeff_eval_nat_zero(coeff_tail(coeff_tail(Polynomial[R].monomial(Nat.1, r).coeff)), x)
    coeff_eval(coeff_tail(coeff_tail(Polynomial[R].monomial(Nat.1, r).coeff)), x, Nat.0) = R.0
    coeff_eval(Polynomial[R].monomial(Nat.1, r).coeff, x, Nat.0.suc.suc) =
        R.0 + x * (r + x * R.0)
    R.0 + x * (r + x * R.0) = r * x
    polynomial_eval_bound(Polynomial[R].monomial(Nat.1, r), x, Nat.0.suc.suc) = r * x
    polynomial_eval(Polynomial[R].monomial(Nat.1, r), x) = r * x
}

/// Evaluation of a monomial of degree two at a point is a scalar square.
theorem monomial_two_eval[R: CommRing](r: R, x: R) {
    polynomial_eval(Polynomial[R].monomial(Nat.2, r), x) = r * x * x
} by {
    polynomial_monomial_support_bounded_by_suc(Nat.2, r)
    polynomial_support_bounded_by(Polynomial[R].monomial(Nat.2, r), Nat.0.suc.suc.suc)
    polynomial_eval_eq_eval_bound_of_support_bounded(
        Polynomial[R].monomial(Nat.2, r), x, Nat.0.suc.suc.suc)
    polynomial_eval(Polynomial[R].monomial(Nat.2, r), x) =
        polynomial_eval_bound(Polynomial[R].monomial(Nat.2, r), x, Nat.0.suc.suc.suc)
    polynomial_eval_bound_eq_coeff_eval(Polynomial[R].monomial(Nat.2, r), x, Nat.0.suc.suc.suc)
    coeff_eval(Polynomial[R].monomial(Nat.2, r).coeff, x, Nat.0.suc.suc.suc) =
        Polynomial[R].monomial(Nat.2, r).coeff(Nat.0) +
        x * coeff_eval(coeff_tail(Polynomial[R].monomial(Nat.2, r).coeff), x, Nat.0.suc.suc)
    polynomial_monomial_coeff_of_ne(Nat.2, r, Nat.0)
    Nat.0 != Nat.2
    Polynomial[R].monomial(Nat.2, r).coeff(Nat.0) = R.0
    coeff_eval(coeff_tail(Polynomial[R].monomial(Nat.2, r).coeff), x, Nat.0.suc.suc) =
        Polynomial[R].monomial(Nat.2, r).coeff(Nat.1) +
        x * coeff_eval(coeff_tail(coeff_tail(Polynomial[R].monomial(Nat.2, r).coeff)), x, Nat.0.suc)
    polynomial_monomial_coeff_of_ne(Nat.2, r, Nat.1)
    Nat.1 != Nat.2
    Polynomial[R].monomial(Nat.2, r).coeff(Nat.1) = R.0
    coeff_tail(coeff_tail(Polynomial[R].monomial(Nat.2, r).coeff))(Nat.0) =
        Polynomial[R].monomial(Nat.2, r).coeff(Nat.2)
    coeff_eval(coeff_tail(coeff_tail(Polynomial[R].monomial(Nat.2, r).coeff)), x, Nat.0.suc) =
        Polynomial[R].monomial(Nat.2, r).coeff(Nat.2) +
        x * coeff_eval(coeff_tail(coeff_tail(coeff_tail(
            Polynomial[R].monomial(Nat.2, r).coeff))), x, Nat.0)
    Polynomial[R].monomial(Nat.2, r).coeff(Nat.2) = r
    ring_coeff_eval_nat_zero(coeff_tail(coeff_tail(coeff_tail(
        Polynomial[R].monomial(Nat.2, r).coeff))), x)
    coeff_eval(coeff_tail(coeff_tail(coeff_tail(
        Polynomial[R].monomial(Nat.2, r).coeff))), x, Nat.0) = R.0
    coeff_eval(Polynomial[R].monomial(Nat.2, r).coeff, x, Nat.0.suc.suc.suc) =
        R.0 + x * (R.0 + x * (r + x * R.0))
    R.0 + x * (R.0 + x * (r + x * R.0)) = r * x * x
    polynomial_eval_bound(Polynomial[R].monomial(Nat.2, r), x, Nat.0.suc.suc.suc) = r * x * x
    polynomial_eval(Polynomial[R].monomial(Nat.2, r), x) = r * x * x
}

/// Evaluation of a linear polynomial at a point.
theorem linear_polynomial_eval[R: CommRing](a: R, b: R, x: R) {
    polynomial_eval(linear_polynomial(a, b), x) = b + a * x
} by {
    polynomial_eval(linear_polynomial(a, b), x) =
        polynomial_eval(polynomial_constant(b) + polynomial_monomial(Nat.1, a), x)
    polynomial_eval_add(polynomial_constant(b), polynomial_monomial(Nat.1, a), x)
    polynomial_eval(polynomial_constant(b) + polynomial_monomial(Nat.1, a), x) =
        polynomial_eval(polynomial_constant(b), x) +
        polynomial_eval(polynomial_monomial(Nat.1, a), x)
    polynomial_eval_constant(b, x)
    polynomial_eval(polynomial_constant(b), x) = b
    monomial_one_eval(a, x)
    polynomial_eval(polynomial_monomial(Nat.1, a), x) = a * x
    polynomial_eval(polynomial_constant(b) + polynomial_monomial(Nat.1, a), x) = b + a * x
    polynomial_eval(linear_polynomial(a, b), x) = b + a * x
}

/// Evaluation of a quadratic polynomial at a point.
theorem quadratic_polynomial_eval[R: CommRing](a: R, b: R, c: R, x: R) {
    polynomial_eval(quadratic_polynomial(a, b, c), x) = c + b * x + a * x * x
} by {
    polynomial_eval(quadratic_polynomial(a, b, c), x) =
        polynomial_eval(polynomial_constant(c) + polynomial_monomial(Nat.1, b) +
            polynomial_monomial(Nat.2, a), x)
    polynomial_eval_add(polynomial_constant(c) + polynomial_monomial(Nat.1, b),
        polynomial_monomial(Nat.2, a), x)
    polynomial_eval(polynomial_constant(c) + polynomial_monomial(Nat.1, b) +
        polynomial_monomial(Nat.2, a), x) =
        polynomial_eval(polynomial_constant(c) + polynomial_monomial(Nat.1, b), x) +
        polynomial_eval(polynomial_monomial(Nat.2, a), x)
    polynomial_eval_add(polynomial_constant(c), polynomial_monomial(Nat.1, b), x)
    polynomial_eval(polynomial_constant(c) + polynomial_monomial(Nat.1, b), x) =
        polynomial_eval(polynomial_constant(c), x) +
        polynomial_eval(polynomial_monomial(Nat.1, b), x)
    polynomial_eval_constant(c, x)
    polynomial_eval(polynomial_constant(c), x) = c
    monomial_one_eval(b, x)
    polynomial_eval(polynomial_monomial(Nat.1, b), x) = b * x
    monomial_two_eval(a, x)
    polynomial_eval(polynomial_monomial(Nat.2, a), x) = a * x * x
    polynomial_eval(polynomial_constant(c) + polynomial_monomial(Nat.1, b) +
        polynomial_monomial(Nat.2, a), x) = c + b * x + a * x * x
    polynomial_eval(quadratic_polynomial(a, b, c), x) = c + b * x + a * x * x
}

// ---------------------------------------------------------------------------
// Coefficients of products of linear and quadratic polynomials
// ---------------------------------------------------------------------------

/// The convolution coefficient is the range sum of the convolution summand.
lemma ring_mul_coeff_eq_range_sum[R: CommRing](p: Polynomial[R], q: Polynomial[R], k: Nat) {
    polynomial_mul_coeff(p, q, k) = range_sum(polynomial_mul_term_coeff(p, q, k), k.suc)
} by {
    forall(i: Nat) {
        (function(j: Nat) { polynomial_mul_term_coeff(p, q, k, j) })(i) =
            polynomial_mul_term_coeff(p, q, k)(i)
    }
    function_extensionality[Nat, R](
        function(j: Nat) { polynomial_mul_term_coeff(p, q, k, j) },
        polynomial_mul_term_coeff(p, q, k))
    function(j: Nat) { polynomial_mul_term_coeff(p, q, k, j) } =
        polynomial_mul_term_coeff(p, q, k)
    polynomial_mul_coeff(p, q, k) =
        partial(function(j: Nat) { polynomial_mul_term_coeff(p, q, k, j) }, k.suc)
    polynomial_mul_coeff(p, q, k) = partial(polynomial_mul_term_coeff(p, q, k), k.suc)
    range_sum_eq_partial(polynomial_mul_term_coeff(p, q, k), k.suc)
    range_sum(polynomial_mul_term_coeff(p, q, k), k.suc) =
        partial(polynomial_mul_term_coeff(p, q, k), k.suc)
    polynomial_mul_coeff(p, q, k) = range_sum(polynomial_mul_term_coeff(p, q, k), k.suc)
}

/// The convolution summand at one index.
theorem ring_mul_term_coeff_at[R: Semiring](p: Polynomial[R], q: Polynomial[R], k: Nat, i: Nat) {
    polynomial_mul_term_coeff(p, q, k, i) = p.coeff(i) * q.coeff(k - i)
}

/// Three is at most four.
lemma ring_nat_three_le_four {
    Nat.3 <= Nat.4
} by {
    lt_suc(Nat.3)
    Nat.3 < Nat.3.suc
    Nat.3.suc = Nat.4
    lt_imp_lte[Nat](Nat.3, Nat.4)
    Nat.3 <= Nat.4
}

/// Three is at most five.
lemma ring_nat_three_le_five {
    Nat.3 <= Nat.5
} by {
    ring_nat_three_le_four
    Nat.3 <= Nat.4
    lt_suc(Nat.4)
    Nat.4 < Nat.4.suc
    Nat.4.suc = Nat.5
    lt_imp_lte[Nat](Nat.4, Nat.5)
    Nat.4 <= Nat.5
    lte_trans(Nat.3, Nat.4, Nat.5)
    Nat.3 <= Nat.5
}

/// The constant coefficient of a product of linear polynomials is the product
/// of the constant coefficients.
theorem linear_mul_coeff_zero[R: CommRing](a: R, b: R, c: R, d: R) {
    polynomial_mul(linear_polynomial(a, b), linear_polynomial(c, d)).coeff(Nat.0) = b * d
} by {
    polynomial_mul_coeff_apply(linear_polynomial(a, b), linear_polynomial(c, d), Nat.0)
    polynomial_mul(linear_polynomial(a, b), linear_polynomial(c, d)).coeff(Nat.0) =
        polynomial_mul_coeff(linear_polynomial(a, b), linear_polynomial(c, d), Nat.0)
    ring_mul_coeff_eq_range_sum(linear_polynomial(a, b), linear_polynomial(c, d), Nat.0)
    polynomial_mul_coeff(linear_polynomial(a, b), linear_polynomial(c, d), Nat.0) =
        range_sum(polynomial_mul_term_coeff(linear_polynomial(a, b), linear_polynomial(c, d), Nat.0), Nat.1)
    range_sum_one(polynomial_mul_term_coeff(linear_polynomial(a, b), linear_polynomial(c, d), Nat.0))
    range_sum(polynomial_mul_term_coeff(linear_polynomial(a, b), linear_polynomial(c, d), Nat.0), Nat.1) =
        polynomial_mul_term_coeff(linear_polynomial(a, b), linear_polynomial(c, d), Nat.0, Nat.0)
    ring_mul_term_coeff_at(linear_polynomial(a, b), linear_polynomial(c, d), Nat.0, Nat.0)
    polynomial_mul_term_coeff(linear_polynomial(a, b), linear_polynomial(c, d), Nat.0, Nat.0) =
        linear_polynomial(a, b).coeff(Nat.0) * linear_polynomial(c, d).coeff(Nat.0 - Nat.0)
    ring_nat_sub_eq_of_add(Nat.0, Nat.0, Nat.0)
    Nat.0 + Nat.0 = Nat.0
    Nat.0 - Nat.0 = Nat.0
    linear_polynomial_coeff_zero(a, b)
    linear_polynomial(a, b).coeff(Nat.0) = b
    linear_polynomial_coeff_zero(c, d)
    linear_polynomial(c, d).coeff(Nat.0) = d
    polynomial_mul_term_coeff(linear_polynomial(a, b), linear_polynomial(c, d), Nat.0, Nat.0) = b * d
    range_sum(polynomial_mul_term_coeff(linear_polynomial(a, b), linear_polynomial(c, d), Nat.0), Nat.1) = b * d
    polynomial_mul_coeff(linear_polynomial(a, b), linear_polynomial(c, d), Nat.0) = b * d
    polynomial_mul(linear_polynomial(a, b), linear_polynomial(c, d)).coeff(Nat.0) = b * d
}

/// The linear coefficient of a product of linear polynomials.
theorem linear_mul_coeff_one[R: CommRing](a: R, b: R, c: R, d: R) {
    polynomial_mul(linear_polynomial(a, b), linear_polynomial(c, d)).coeff(Nat.1) = a * d + b * c
} by {
    polynomial_mul_coeff_apply(linear_polynomial(a, b), linear_polynomial(c, d), Nat.1)
    polynomial_mul(linear_polynomial(a, b), linear_polynomial(c, d)).coeff(Nat.1) =
        polynomial_mul_coeff(linear_polynomial(a, b), linear_polynomial(c, d), Nat.1)
    ring_mul_coeff_eq_range_sum(linear_polynomial(a, b), linear_polynomial(c, d), Nat.1)
    polynomial_mul_coeff(linear_polynomial(a, b), linear_polynomial(c, d), Nat.1) =
        range_sum(polynomial_mul_term_coeff(linear_polynomial(a, b), linear_polynomial(c, d), Nat.1), Nat.2)
    range_sum_suc(polynomial_mul_term_coeff(linear_polynomial(a, b), linear_polynomial(c, d), Nat.1), Nat.1)
    range_sum(polynomial_mul_term_coeff(linear_polynomial(a, b), linear_polynomial(c, d), Nat.1), Nat.2) =
        range_sum(polynomial_mul_term_coeff(linear_polynomial(a, b), linear_polynomial(c, d), Nat.1), Nat.1) +
        polynomial_mul_term_coeff(linear_polynomial(a, b), linear_polynomial(c, d), Nat.1, Nat.1)
    range_sum_one(polynomial_mul_term_coeff(linear_polynomial(a, b), linear_polynomial(c, d), Nat.1))
    range_sum(polynomial_mul_term_coeff(linear_polynomial(a, b), linear_polynomial(c, d), Nat.1), Nat.1) =
        polynomial_mul_term_coeff(linear_polynomial(a, b), linear_polynomial(c, d), Nat.1, Nat.0)
    ring_mul_term_coeff_at(linear_polynomial(a, b), linear_polynomial(c, d), Nat.1, Nat.0)
    polynomial_mul_term_coeff(linear_polynomial(a, b), linear_polynomial(c, d), Nat.1, Nat.0) =
        linear_polynomial(a, b).coeff(Nat.0) * linear_polynomial(c, d).coeff(Nat.1 - Nat.0)
    ring_nat_sub_eq_of_add(Nat.1, Nat.0, Nat.1)
    Nat.1 + Nat.0 = Nat.1
    Nat.1 - Nat.0 = Nat.1
    linear_polynomial_coeff_zero(a, b)
    linear_polynomial(a, b).coeff(Nat.0) = b
    linear_polynomial_coeff_one(c, d)
    linear_polynomial(c, d).coeff(Nat.1) = c
    polynomial_mul_term_coeff(linear_polynomial(a, b), linear_polynomial(c, d), Nat.1, Nat.0) = b * c
    ring_mul_term_coeff_at(linear_polynomial(a, b), linear_polynomial(c, d), Nat.1, Nat.1)
    polynomial_mul_term_coeff(linear_polynomial(a, b), linear_polynomial(c, d), Nat.1, Nat.1) =
        linear_polynomial(a, b).coeff(Nat.1) * linear_polynomial(c, d).coeff(Nat.1 - Nat.1)
    ring_nat_sub_eq_of_add(Nat.1, Nat.1, Nat.0)
    Nat.0 + Nat.1 = Nat.1
    Nat.1 - Nat.1 = Nat.0
    linear_polynomial_coeff_one(a, b)
    linear_polynomial(a, b).coeff(Nat.1) = a
    linear_polynomial_coeff_zero(c, d)
    linear_polynomial(c, d).coeff(Nat.0) = d
    polynomial_mul_term_coeff(linear_polynomial(a, b), linear_polynomial(c, d), Nat.1, Nat.1) = a * d
    range_sum(polynomial_mul_term_coeff(linear_polynomial(a, b), linear_polynomial(c, d), Nat.1), Nat.1) = b * c
    range_sum(polynomial_mul_term_coeff(linear_polynomial(a, b), linear_polynomial(c, d), Nat.1), Nat.2) = b * c + a * d
    b * c + a * d = a * d + b * c
    polynomial_mul_coeff(linear_polynomial(a, b), linear_polynomial(c, d), Nat.1) = a * d + b * c
    polynomial_mul(linear_polynomial(a, b), linear_polynomial(c, d)).coeff(Nat.1) = a * d + b * c
}

/// The quadratic coefficient of a product of linear polynomials is the product
/// of the linear coefficients.
theorem linear_mul_coeff_two[R: CommRing](a: R, b: R, c: R, d: R) {
    polynomial_mul(linear_polynomial(a, b), linear_polynomial(c, d)).coeff(Nat.2) = a * c
} by {
    polynomial_mul_coeff_apply(linear_polynomial(a, b), linear_polynomial(c, d), Nat.2)
    polynomial_mul(linear_polynomial(a, b), linear_polynomial(c, d)).coeff(Nat.2) =
        polynomial_mul_coeff(linear_polynomial(a, b), linear_polynomial(c, d), Nat.2)
    ring_mul_coeff_eq_range_sum(linear_polynomial(a, b), linear_polynomial(c, d), Nat.2)
    polynomial_mul_coeff(linear_polynomial(a, b), linear_polynomial(c, d), Nat.2) =
        range_sum(polynomial_mul_term_coeff(linear_polynomial(a, b), linear_polynomial(c, d), Nat.2), Nat.3)
    range_sum_suc(polynomial_mul_term_coeff(linear_polynomial(a, b), linear_polynomial(c, d), Nat.2), Nat.2)
    range_sum(polynomial_mul_term_coeff(linear_polynomial(a, b), linear_polynomial(c, d), Nat.2), Nat.3) =
        range_sum(polynomial_mul_term_coeff(linear_polynomial(a, b), linear_polynomial(c, d), Nat.2), Nat.2) +
        polynomial_mul_term_coeff(linear_polynomial(a, b), linear_polynomial(c, d), Nat.2, Nat.2)
    range_sum_suc(polynomial_mul_term_coeff(linear_polynomial(a, b), linear_polynomial(c, d), Nat.2), Nat.1)
    range_sum(polynomial_mul_term_coeff(linear_polynomial(a, b), linear_polynomial(c, d), Nat.2), Nat.2) =
        range_sum(polynomial_mul_term_coeff(linear_polynomial(a, b), linear_polynomial(c, d), Nat.2), Nat.1) +
        polynomial_mul_term_coeff(linear_polynomial(a, b), linear_polynomial(c, d), Nat.2, Nat.1)
    range_sum_one(polynomial_mul_term_coeff(linear_polynomial(a, b), linear_polynomial(c, d), Nat.2))
    range_sum(polynomial_mul_term_coeff(linear_polynomial(a, b), linear_polynomial(c, d), Nat.2), Nat.1) =
        polynomial_mul_term_coeff(linear_polynomial(a, b), linear_polynomial(c, d), Nat.2, Nat.0)
    ring_mul_term_coeff_at(linear_polynomial(a, b), linear_polynomial(c, d), Nat.2, Nat.0)
    polynomial_mul_term_coeff(linear_polynomial(a, b), linear_polynomial(c, d), Nat.2, Nat.0) =
        linear_polynomial(a, b).coeff(Nat.0) * linear_polynomial(c, d).coeff(Nat.2 - Nat.0)
    ring_nat_sub_eq_of_add(Nat.2, Nat.0, Nat.2)
    Nat.2 + Nat.0 = Nat.2
    Nat.2 - Nat.0 = Nat.2
    linear_polynomial_coeff_zero(a, b)
    linear_polynomial(a, b).coeff(Nat.0) = b
    linear_polynomial_support_bounded_by_two(c, d)
    polynomial_support_bounded_by(linear_polynomial(c, d), Nat.2)
    lt_not_ref(Nat.2)
    not Nat.2 < Nat.2
    polynomial_support_bounded_by_apply(linear_polynomial(c, d), Nat.2, Nat.2)
    linear_polynomial(c, d).coeff(Nat.2) = R.0
    polynomial_mul_term_coeff(linear_polynomial(a, b), linear_polynomial(c, d), Nat.2, Nat.0) = b * R.0
    b * R.0 = R.0
    polynomial_mul_term_coeff(linear_polynomial(a, b), linear_polynomial(c, d), Nat.2, Nat.0) = R.0
    ring_mul_term_coeff_at(linear_polynomial(a, b), linear_polynomial(c, d), Nat.2, Nat.1)
    polynomial_mul_term_coeff(linear_polynomial(a, b), linear_polynomial(c, d), Nat.2, Nat.1) =
        linear_polynomial(a, b).coeff(Nat.1) * linear_polynomial(c, d).coeff(Nat.2 - Nat.1)
    ring_nat_sub_eq_of_add(Nat.2, Nat.1, Nat.1)
    Nat.1 + Nat.1 = Nat.2
    Nat.2 - Nat.1 = Nat.1
    linear_polynomial_coeff_one(a, b)
    linear_polynomial(a, b).coeff(Nat.1) = a
    linear_polynomial_coeff_one(c, d)
    linear_polynomial(c, d).coeff(Nat.1) = c
    polynomial_mul_term_coeff(linear_polynomial(a, b), linear_polynomial(c, d), Nat.2, Nat.1) = a * c
    ring_mul_term_coeff_at(linear_polynomial(a, b), linear_polynomial(c, d), Nat.2, Nat.2)
    polynomial_mul_term_coeff(linear_polynomial(a, b), linear_polynomial(c, d), Nat.2, Nat.2) =
        linear_polynomial(a, b).coeff(Nat.2) * linear_polynomial(c, d).coeff(Nat.2 - Nat.2)
    ring_nat_sub_eq_of_add(Nat.2, Nat.2, Nat.0)
    Nat.0 + Nat.2 = Nat.2
    Nat.2 - Nat.2 = Nat.0
    linear_polynomial_support_bounded_by_two(a, b)
    polynomial_support_bounded_by(linear_polynomial(a, b), Nat.2)
    polynomial_support_bounded_by_apply(linear_polynomial(a, b), Nat.2, Nat.2)
    linear_polynomial(a, b).coeff(Nat.2) = R.0
    linear_polynomial_coeff_zero(c, d)
    linear_polynomial(c, d).coeff(Nat.0) = d
    polynomial_mul_term_coeff(linear_polynomial(a, b), linear_polynomial(c, d), Nat.2, Nat.2) = R.0 * d
    R.0 * d = R.0
    polynomial_mul_term_coeff(linear_polynomial(a, b), linear_polynomial(c, d), Nat.2, Nat.2) = R.0
    range_sum(polynomial_mul_term_coeff(linear_polynomial(a, b), linear_polynomial(c, d), Nat.2), Nat.1) = R.0
    range_sum(polynomial_mul_term_coeff(linear_polynomial(a, b), linear_polynomial(c, d), Nat.2), Nat.2) = R.0 + a * c
    range_sum(polynomial_mul_term_coeff(linear_polynomial(a, b), linear_polynomial(c, d), Nat.2), Nat.3) = R.0 + a * c + R.0
    R.0 + a * c + R.0 = a * c
    polynomial_mul_coeff(linear_polynomial(a, b), linear_polynomial(c, d), Nat.2) = a * c
    polynomial_mul(linear_polynomial(a, b), linear_polynomial(c, d)).coeff(Nat.2) = a * c
}

/// The cubic coefficient of a product of linear polynomials vanishes.
theorem linear_mul_coeff_three[R: CommRing](a: R, b: R, c: R, d: R) {
    polynomial_mul(linear_polynomial(a, b), linear_polynomial(c, d)).coeff(Nat.3) = R.0
} by {
    polynomial_mul_coeff_apply(linear_polynomial(a, b), linear_polynomial(c, d), Nat.3)
    polynomial_mul(linear_polynomial(a, b), linear_polynomial(c, d)).coeff(Nat.3) =
        polynomial_mul_coeff(linear_polynomial(a, b), linear_polynomial(c, d), Nat.3)
    ring_mul_coeff_eq_range_sum(linear_polynomial(a, b), linear_polynomial(c, d), Nat.3)
    polynomial_mul_coeff(linear_polynomial(a, b), linear_polynomial(c, d), Nat.3) =
        range_sum(polynomial_mul_term_coeff(linear_polynomial(a, b), linear_polynomial(c, d), Nat.3), Nat.4)
    range_sum_suc(polynomial_mul_term_coeff(linear_polynomial(a, b), linear_polynomial(c, d), Nat.3), Nat.3)
    range_sum(polynomial_mul_term_coeff(linear_polynomial(a, b), linear_polynomial(c, d), Nat.3), Nat.4) =
        range_sum(polynomial_mul_term_coeff(linear_polynomial(a, b), linear_polynomial(c, d), Nat.3), Nat.3) +
        polynomial_mul_term_coeff(linear_polynomial(a, b), linear_polynomial(c, d), Nat.3, Nat.3)
    range_sum_suc(polynomial_mul_term_coeff(linear_polynomial(a, b), linear_polynomial(c, d), Nat.3), Nat.2)
    range_sum(polynomial_mul_term_coeff(linear_polynomial(a, b), linear_polynomial(c, d), Nat.3), Nat.3) =
        range_sum(polynomial_mul_term_coeff(linear_polynomial(a, b), linear_polynomial(c, d), Nat.3), Nat.2) +
        polynomial_mul_term_coeff(linear_polynomial(a, b), linear_polynomial(c, d), Nat.3, Nat.2)
    range_sum_suc(polynomial_mul_term_coeff(linear_polynomial(a, b), linear_polynomial(c, d), Nat.3), Nat.1)
    range_sum(polynomial_mul_term_coeff(linear_polynomial(a, b), linear_polynomial(c, d), Nat.3), Nat.2) =
        range_sum(polynomial_mul_term_coeff(linear_polynomial(a, b), linear_polynomial(c, d), Nat.3), Nat.1) +
        polynomial_mul_term_coeff(linear_polynomial(a, b), linear_polynomial(c, d), Nat.3, Nat.1)
    range_sum_one(polynomial_mul_term_coeff(linear_polynomial(a, b), linear_polynomial(c, d), Nat.3))
    range_sum(polynomial_mul_term_coeff(linear_polynomial(a, b), linear_polynomial(c, d), Nat.3), Nat.1) =
        polynomial_mul_term_coeff(linear_polynomial(a, b), linear_polynomial(c, d), Nat.3, Nat.0)
    ring_mul_term_coeff_at(linear_polynomial(a, b), linear_polynomial(c, d), Nat.3, Nat.0)
    polynomial_mul_term_coeff(linear_polynomial(a, b), linear_polynomial(c, d), Nat.3, Nat.0) =
        linear_polynomial(a, b).coeff(Nat.0) * linear_polynomial(c, d).coeff(Nat.3 - Nat.0)
    ring_nat_sub_eq_of_add(Nat.3, Nat.0, Nat.3)
    Nat.3 + Nat.0 = Nat.3
    Nat.3 - Nat.0 = Nat.3
    linear_polynomial_coeff_zero(a, b)
    linear_polynomial(a, b).coeff(Nat.0) = b
    linear_polynomial_support_bounded_by_two(c, d)
    polynomial_support_bounded_by(linear_polynomial(c, d), Nat.2)
    ring_nat_two_le_three
    Nat.2 <= Nat.3
    lte_imp_not_lt(Nat.2, Nat.3)
    not Nat.3 < Nat.2
    polynomial_support_bounded_by_apply(linear_polynomial(c, d), Nat.2, Nat.3)
    linear_polynomial(c, d).coeff(Nat.3) = R.0
    polynomial_mul_term_coeff(linear_polynomial(a, b), linear_polynomial(c, d), Nat.3, Nat.0) = b * R.0
    b * R.0 = R.0
    polynomial_mul_term_coeff(linear_polynomial(a, b), linear_polynomial(c, d), Nat.3, Nat.0) = R.0
    ring_mul_term_coeff_at(linear_polynomial(a, b), linear_polynomial(c, d), Nat.3, Nat.1)
    polynomial_mul_term_coeff(linear_polynomial(a, b), linear_polynomial(c, d), Nat.3, Nat.1) =
        linear_polynomial(a, b).coeff(Nat.1) * linear_polynomial(c, d).coeff(Nat.3 - Nat.1)
    ring_nat_sub_eq_of_add(Nat.3, Nat.1, Nat.2)
    Nat.2 + Nat.1 = Nat.3
    Nat.3 - Nat.1 = Nat.2
    linear_polynomial_coeff_one(a, b)
    linear_polynomial(a, b).coeff(Nat.1) = a
    polynomial_support_bounded_by_apply(linear_polynomial(c, d), Nat.2, Nat.2)
    linear_polynomial(c, d).coeff(Nat.2) = R.0
    polynomial_mul_term_coeff(linear_polynomial(a, b), linear_polynomial(c, d), Nat.3, Nat.1) = a * R.0
    a * R.0 = R.0
    polynomial_mul_term_coeff(linear_polynomial(a, b), linear_polynomial(c, d), Nat.3, Nat.1) = R.0
    ring_mul_term_coeff_at(linear_polynomial(a, b), linear_polynomial(c, d), Nat.3, Nat.2)
    polynomial_mul_term_coeff(linear_polynomial(a, b), linear_polynomial(c, d), Nat.3, Nat.2) =
        linear_polynomial(a, b).coeff(Nat.2) * linear_polynomial(c, d).coeff(Nat.3 - Nat.2)
    ring_nat_sub_eq_of_add(Nat.3, Nat.2, Nat.1)
    Nat.1 + Nat.2 = Nat.3
    Nat.3 - Nat.2 = Nat.1
    polynomial_support_bounded_by_apply(linear_polynomial(a, b), Nat.2, Nat.2)
    linear_polynomial(a, b).coeff(Nat.2) = R.0
    linear_polynomial_coeff_one(c, d)
    linear_polynomial(c, d).coeff(Nat.1) = c
    polynomial_mul_term_coeff(linear_polynomial(a, b), linear_polynomial(c, d), Nat.3, Nat.2) = R.0 * c
    R.0 * c = R.0
    polynomial_mul_term_coeff(linear_polynomial(a, b), linear_polynomial(c, d), Nat.3, Nat.2) = R.0
    ring_mul_term_coeff_at(linear_polynomial(a, b), linear_polynomial(c, d), Nat.3, Nat.3)
    polynomial_mul_term_coeff(linear_polynomial(a, b), linear_polynomial(c, d), Nat.3, Nat.3) =
        linear_polynomial(a, b).coeff(Nat.3) * linear_polynomial(c, d).coeff(Nat.3 - Nat.3)
    ring_nat_sub_eq_of_add(Nat.3, Nat.3, Nat.0)
    Nat.0 + Nat.3 = Nat.3
    Nat.3 - Nat.3 = Nat.0
    polynomial_support_bounded_by_apply(linear_polynomial(a, b), Nat.2, Nat.3)
    linear_polynomial(a, b).coeff(Nat.3) = R.0
    linear_polynomial_coeff_zero(c, d)
    linear_polynomial(c, d).coeff(Nat.0) = d
    polynomial_mul_term_coeff(linear_polynomial(a, b), linear_polynomial(c, d), Nat.3, Nat.3) = R.0 * d
    R.0 * d = R.0
    polynomial_mul_term_coeff(linear_polynomial(a, b), linear_polynomial(c, d), Nat.3, Nat.3) = R.0
    range_sum(polynomial_mul_term_coeff(linear_polynomial(a, b), linear_polynomial(c, d), Nat.3), Nat.1) = R.0
    range_sum(polynomial_mul_term_coeff(linear_polynomial(a, b), linear_polynomial(c, d), Nat.3), Nat.2) = R.0 + R.0
    range_sum(polynomial_mul_term_coeff(linear_polynomial(a, b), linear_polynomial(c, d), Nat.3), Nat.3) = R.0 + R.0 + R.0
    range_sum(polynomial_mul_term_coeff(linear_polynomial(a, b), linear_polynomial(c, d), Nat.3), Nat.4) = R.0 + R.0 + R.0 + R.0
    R.0 + R.0 + R.0 + R.0 = R.0
    polynomial_mul_coeff(linear_polynomial(a, b), linear_polynomial(c, d), Nat.3) = R.0
    polynomial_mul(linear_polynomial(a, b), linear_polynomial(c, d)).coeff(Nat.3) = R.0
}

/// The product of two linear polynomials is supported below four coefficient
/// slots.
theorem linear_mul_support_bounded_by_four[R: CommRing](a: R, b: R, c: R, d: R) {
    polynomial_support_bounded_by(polynomial_mul(linear_polynomial(a, b), linear_polynomial(c, d)), Nat.4)
} by {
    linear_polynomial_support_bounded_by_two(a, b)
    polynomial_support_bounded_by(linear_polynomial(a, b), Nat.2)
    linear_polynomial_support_bounded_by_two(c, d)
    polynomial_support_bounded_by(linear_polynomial(c, d), Nat.2)
    polynomial_mul_support_bounded_by_add(linear_polynomial(a, b), linear_polynomial(c, d), Nat.2, Nat.2)
    polynomial_support_bounded_by(polynomial_mul(linear_polynomial(a, b), linear_polynomial(c, d)), Nat.2 + Nat.2)
    polynomial_support_bounded_by(polynomial_mul(linear_polynomial(a, b), linear_polynomial(c, d)), Nat.4)
}

/// The quartic coefficient of a product of quadratics is the product of the
/// leading coefficients.
theorem quadratic_mul_coeff_four[R: CommRing](a: R, b: R, c: R, d: R, e: R, g: R) {
    polynomial_mul(quadratic_polynomial(a, b, c), quadratic_polynomial(d, e, g)).coeff(Nat.4) = a * d
} by {
    polynomial_mul_coeff_apply(quadratic_polynomial(a, b, c), quadratic_polynomial(d, e, g), Nat.4)
    polynomial_mul(quadratic_polynomial(a, b, c), quadratic_polynomial(d, e, g)).coeff(Nat.4) =
        polynomial_mul_coeff(quadratic_polynomial(a, b, c), quadratic_polynomial(d, e, g), Nat.4)
    ring_mul_coeff_eq_range_sum(quadratic_polynomial(a, b, c), quadratic_polynomial(d, e, g), Nat.4)
    polynomial_mul_coeff(quadratic_polynomial(a, b, c), quadratic_polynomial(d, e, g), Nat.4) =
        range_sum(polynomial_mul_term_coeff(quadratic_polynomial(a, b, c), quadratic_polynomial(d, e, g), Nat.4), Nat.5)
    range_sum_suc(polynomial_mul_term_coeff(quadratic_polynomial(a, b, c), quadratic_polynomial(d, e, g), Nat.4), Nat.4)
    range_sum(polynomial_mul_term_coeff(quadratic_polynomial(a, b, c), quadratic_polynomial(d, e, g), Nat.4), Nat.5) =
        range_sum(polynomial_mul_term_coeff(quadratic_polynomial(a, b, c), quadratic_polynomial(d, e, g), Nat.4), Nat.4) +
        polynomial_mul_term_coeff(quadratic_polynomial(a, b, c), quadratic_polynomial(d, e, g), Nat.4, Nat.4)
    range_sum_suc(polynomial_mul_term_coeff(quadratic_polynomial(a, b, c), quadratic_polynomial(d, e, g), Nat.4), Nat.3)
    range_sum(polynomial_mul_term_coeff(quadratic_polynomial(a, b, c), quadratic_polynomial(d, e, g), Nat.4), Nat.4) =
        range_sum(polynomial_mul_term_coeff(quadratic_polynomial(a, b, c), quadratic_polynomial(d, e, g), Nat.4), Nat.3) +
        polynomial_mul_term_coeff(quadratic_polynomial(a, b, c), quadratic_polynomial(d, e, g), Nat.4, Nat.3)
    range_sum_suc(polynomial_mul_term_coeff(quadratic_polynomial(a, b, c), quadratic_polynomial(d, e, g), Nat.4), Nat.2)
    range_sum(polynomial_mul_term_coeff(quadratic_polynomial(a, b, c), quadratic_polynomial(d, e, g), Nat.4), Nat.3) =
        range_sum(polynomial_mul_term_coeff(quadratic_polynomial(a, b, c), quadratic_polynomial(d, e, g), Nat.4), Nat.2) +
        polynomial_mul_term_coeff(quadratic_polynomial(a, b, c), quadratic_polynomial(d, e, g), Nat.4, Nat.2)
    range_sum_suc(polynomial_mul_term_coeff(quadratic_polynomial(a, b, c), quadratic_polynomial(d, e, g), Nat.4), Nat.1)
    range_sum(polynomial_mul_term_coeff(quadratic_polynomial(a, b, c), quadratic_polynomial(d, e, g), Nat.4), Nat.2) =
        range_sum(polynomial_mul_term_coeff(quadratic_polynomial(a, b, c), quadratic_polynomial(d, e, g), Nat.4), Nat.1) +
        polynomial_mul_term_coeff(quadratic_polynomial(a, b, c), quadratic_polynomial(d, e, g), Nat.4, Nat.1)
    range_sum_one(polynomial_mul_term_coeff(quadratic_polynomial(a, b, c), quadratic_polynomial(d, e, g), Nat.4))
    range_sum(polynomial_mul_term_coeff(quadratic_polynomial(a, b, c), quadratic_polynomial(d, e, g), Nat.4), Nat.1) =
        polynomial_mul_term_coeff(quadratic_polynomial(a, b, c), quadratic_polynomial(d, e, g), Nat.4, Nat.0)
    quadratic_polynomial_support_bounded_by_three(d, e, g)
    polynomial_support_bounded_by(quadratic_polynomial(d, e, g), Nat.3)
    quadratic_polynomial_support_bounded_by_three(a, b, c)
    polynomial_support_bounded_by(quadratic_polynomial(a, b, c), Nat.3)
    ring_mul_term_coeff_at(quadratic_polynomial(a, b, c), quadratic_polynomial(d, e, g), Nat.4, Nat.0)
    polynomial_mul_term_coeff(quadratic_polynomial(a, b, c), quadratic_polynomial(d, e, g), Nat.4, Nat.0) =
        quadratic_polynomial(a, b, c).coeff(Nat.0) * quadratic_polynomial(d, e, g).coeff(Nat.4 - Nat.0)
    ring_nat_sub_eq_of_add(Nat.4, Nat.0, Nat.4)
    Nat.4 + Nat.0 = Nat.4
    Nat.4 - Nat.0 = Nat.4
    quadratic_polynomial_coeff_zero(a, b, c)
    quadratic_polynomial(a, b, c).coeff(Nat.0) = c
    ring_nat_three_le_four
    Nat.3 <= Nat.4
    lte_imp_not_lt(Nat.3, Nat.4)
    not Nat.4 < Nat.3
    polynomial_support_bounded_by_apply(quadratic_polynomial(d, e, g), Nat.3, Nat.4)
    quadratic_polynomial(d, e, g).coeff(Nat.4) = R.0
    polynomial_mul_term_coeff(quadratic_polynomial(a, b, c), quadratic_polynomial(d, e, g), Nat.4, Nat.0) = c * R.0
    c * R.0 = R.0
    polynomial_mul_term_coeff(quadratic_polynomial(a, b, c), quadratic_polynomial(d, e, g), Nat.4, Nat.0) = R.0
    ring_mul_term_coeff_at(quadratic_polynomial(a, b, c), quadratic_polynomial(d, e, g), Nat.4, Nat.1)
    polynomial_mul_term_coeff(quadratic_polynomial(a, b, c), quadratic_polynomial(d, e, g), Nat.4, Nat.1) =
        quadratic_polynomial(a, b, c).coeff(Nat.1) * quadratic_polynomial(d, e, g).coeff(Nat.4 - Nat.1)
    ring_nat_sub_eq_of_add(Nat.4, Nat.1, Nat.3)
    Nat.3 + Nat.1 = Nat.4
    Nat.4 - Nat.1 = Nat.3
    quadratic_polynomial_coeff_one(a, b, c)
    quadratic_polynomial(a, b, c).coeff(Nat.1) = b
    lt_not_ref(Nat.3)
    not Nat.3 < Nat.3
    polynomial_support_bounded_by_apply(quadratic_polynomial(d, e, g), Nat.3, Nat.3)
    quadratic_polynomial(d, e, g).coeff(Nat.3) = R.0
    polynomial_mul_term_coeff(quadratic_polynomial(a, b, c), quadratic_polynomial(d, e, g), Nat.4, Nat.1) = b * R.0
    b * R.0 = R.0
    polynomial_mul_term_coeff(quadratic_polynomial(a, b, c), quadratic_polynomial(d, e, g), Nat.4, Nat.1) = R.0
    ring_mul_term_coeff_at(quadratic_polynomial(a, b, c), quadratic_polynomial(d, e, g), Nat.4, Nat.2)
    polynomial_mul_term_coeff(quadratic_polynomial(a, b, c), quadratic_polynomial(d, e, g), Nat.4, Nat.2) =
        quadratic_polynomial(a, b, c).coeff(Nat.2) * quadratic_polynomial(d, e, g).coeff(Nat.4 - Nat.2)
    ring_nat_sub_eq_of_add(Nat.4, Nat.2, Nat.2)
    Nat.2 + Nat.2 = Nat.4
    Nat.4 - Nat.2 = Nat.2
    quadratic_polynomial_coeff_two(a, b, c)
    quadratic_polynomial(a, b, c).coeff(Nat.2) = a
    quadratic_polynomial_coeff_two(d, e, g)
    quadratic_polynomial(d, e, g).coeff(Nat.2) = d
    polynomial_mul_term_coeff(quadratic_polynomial(a, b, c), quadratic_polynomial(d, e, g), Nat.4, Nat.2) = a * d
    ring_mul_term_coeff_at(quadratic_polynomial(a, b, c), quadratic_polynomial(d, e, g), Nat.4, Nat.3)
    polynomial_mul_term_coeff(quadratic_polynomial(a, b, c), quadratic_polynomial(d, e, g), Nat.4, Nat.3) =
        quadratic_polynomial(a, b, c).coeff(Nat.3) * quadratic_polynomial(d, e, g).coeff(Nat.4 - Nat.3)
    ring_nat_sub_eq_of_add(Nat.4, Nat.3, Nat.1)
    Nat.1 + Nat.3 = Nat.4
    Nat.4 - Nat.3 = Nat.1
    polynomial_support_bounded_by_apply(quadratic_polynomial(a, b, c), Nat.3, Nat.3)
    quadratic_polynomial(a, b, c).coeff(Nat.3) = R.0
    quadratic_polynomial_coeff_one(d, e, g)
    quadratic_polynomial(d, e, g).coeff(Nat.1) = e
    polynomial_mul_term_coeff(quadratic_polynomial(a, b, c), quadratic_polynomial(d, e, g), Nat.4, Nat.3) = R.0 * e
    R.0 * e = R.0
    polynomial_mul_term_coeff(quadratic_polynomial(a, b, c), quadratic_polynomial(d, e, g), Nat.4, Nat.3) = R.0
    ring_mul_term_coeff_at(quadratic_polynomial(a, b, c), quadratic_polynomial(d, e, g), Nat.4, Nat.4)
    polynomial_mul_term_coeff(quadratic_polynomial(a, b, c), quadratic_polynomial(d, e, g), Nat.4, Nat.4) =
        quadratic_polynomial(a, b, c).coeff(Nat.4) * quadratic_polynomial(d, e, g).coeff(Nat.4 - Nat.4)
    ring_nat_sub_eq_of_add(Nat.4, Nat.4, Nat.0)
    Nat.0 + Nat.4 = Nat.4
    Nat.4 - Nat.4 = Nat.0
    polynomial_support_bounded_by_apply(quadratic_polynomial(a, b, c), Nat.3, Nat.4)
    quadratic_polynomial(a, b, c).coeff(Nat.4) = R.0
    quadratic_polynomial_coeff_zero(d, e, g)
    quadratic_polynomial(d, e, g).coeff(Nat.0) = g
    polynomial_mul_term_coeff(quadratic_polynomial(a, b, c), quadratic_polynomial(d, e, g), Nat.4, Nat.4) = R.0 * g
    R.0 * g = R.0
    polynomial_mul_term_coeff(quadratic_polynomial(a, b, c), quadratic_polynomial(d, e, g), Nat.4, Nat.4) = R.0
    range_sum(polynomial_mul_term_coeff(quadratic_polynomial(a, b, c), quadratic_polynomial(d, e, g), Nat.4), Nat.1) = R.0
    range_sum(polynomial_mul_term_coeff(quadratic_polynomial(a, b, c), quadratic_polynomial(d, e, g), Nat.4), Nat.2) = R.0 + R.0
    range_sum(polynomial_mul_term_coeff(quadratic_polynomial(a, b, c), quadratic_polynomial(d, e, g), Nat.4), Nat.3) = R.0 + R.0 + a * d
    range_sum(polynomial_mul_term_coeff(quadratic_polynomial(a, b, c), quadratic_polynomial(d, e, g), Nat.4), Nat.4) = R.0 + R.0 + a * d + R.0
    range_sum(polynomial_mul_term_coeff(quadratic_polynomial(a, b, c), quadratic_polynomial(d, e, g), Nat.4), Nat.5) = R.0 + R.0 + a * d + R.0 + R.0
    R.0 + R.0 + a * d + R.0 + R.0 = a * d
    polynomial_mul_coeff(quadratic_polynomial(a, b, c), quadratic_polynomial(d, e, g), Nat.4) = a * d
    polynomial_mul(quadratic_polynomial(a, b, c), quadratic_polynomial(d, e, g)).coeff(Nat.4) = a * d
}

/// The quintic coefficient of a product of quadratics vanishes.
theorem quadratic_mul_coeff_five[R: CommRing](a: R, b: R, c: R, d: R, e: R, g: R) {
    polynomial_mul(quadratic_polynomial(a, b, c), quadratic_polynomial(d, e, g)).coeff(Nat.5) = R.0
} by {
    polynomial_mul_coeff_apply(quadratic_polynomial(a, b, c), quadratic_polynomial(d, e, g), Nat.5)
    polynomial_mul(quadratic_polynomial(a, b, c), quadratic_polynomial(d, e, g)).coeff(Nat.5) =
        polynomial_mul_coeff(quadratic_polynomial(a, b, c), quadratic_polynomial(d, e, g), Nat.5)
    ring_mul_coeff_eq_range_sum(quadratic_polynomial(a, b, c), quadratic_polynomial(d, e, g), Nat.5)
    polynomial_mul_coeff(quadratic_polynomial(a, b, c), quadratic_polynomial(d, e, g), Nat.5) =
        range_sum(polynomial_mul_term_coeff(quadratic_polynomial(a, b, c), quadratic_polynomial(d, e, g), Nat.5), Nat.6)
    range_sum_suc(polynomial_mul_term_coeff(quadratic_polynomial(a, b, c), quadratic_polynomial(d, e, g), Nat.5), Nat.5)
    range_sum(polynomial_mul_term_coeff(quadratic_polynomial(a, b, c), quadratic_polynomial(d, e, g), Nat.5), Nat.6) =
        range_sum(polynomial_mul_term_coeff(quadratic_polynomial(a, b, c), quadratic_polynomial(d, e, g), Nat.5), Nat.5) +
        polynomial_mul_term_coeff(quadratic_polynomial(a, b, c), quadratic_polynomial(d, e, g), Nat.5, Nat.5)
    range_sum_suc(polynomial_mul_term_coeff(quadratic_polynomial(a, b, c), quadratic_polynomial(d, e, g), Nat.5), Nat.4)
    range_sum(polynomial_mul_term_coeff(quadratic_polynomial(a, b, c), quadratic_polynomial(d, e, g), Nat.5), Nat.5) =
        range_sum(polynomial_mul_term_coeff(quadratic_polynomial(a, b, c), quadratic_polynomial(d, e, g), Nat.5), Nat.4) +
        polynomial_mul_term_coeff(quadratic_polynomial(a, b, c), quadratic_polynomial(d, e, g), Nat.5, Nat.4)
    range_sum_suc(polynomial_mul_term_coeff(quadratic_polynomial(a, b, c), quadratic_polynomial(d, e, g), Nat.5), Nat.3)
    range_sum(polynomial_mul_term_coeff(quadratic_polynomial(a, b, c), quadratic_polynomial(d, e, g), Nat.5), Nat.4) =
        range_sum(polynomial_mul_term_coeff(quadratic_polynomial(a, b, c), quadratic_polynomial(d, e, g), Nat.5), Nat.3) +
        polynomial_mul_term_coeff(quadratic_polynomial(a, b, c), quadratic_polynomial(d, e, g), Nat.5, Nat.3)
    range_sum_suc(polynomial_mul_term_coeff(quadratic_polynomial(a, b, c), quadratic_polynomial(d, e, g), Nat.5), Nat.2)
    range_sum(polynomial_mul_term_coeff(quadratic_polynomial(a, b, c), quadratic_polynomial(d, e, g), Nat.5), Nat.3) =
        range_sum(polynomial_mul_term_coeff(quadratic_polynomial(a, b, c), quadratic_polynomial(d, e, g), Nat.5), Nat.2) +
        polynomial_mul_term_coeff(quadratic_polynomial(a, b, c), quadratic_polynomial(d, e, g), Nat.5, Nat.2)
    range_sum_suc(polynomial_mul_term_coeff(quadratic_polynomial(a, b, c), quadratic_polynomial(d, e, g), Nat.5), Nat.1)
    range_sum(polynomial_mul_term_coeff(quadratic_polynomial(a, b, c), quadratic_polynomial(d, e, g), Nat.5), Nat.2) =
        range_sum(polynomial_mul_term_coeff(quadratic_polynomial(a, b, c), quadratic_polynomial(d, e, g), Nat.5), Nat.1) +
        polynomial_mul_term_coeff(quadratic_polynomial(a, b, c), quadratic_polynomial(d, e, g), Nat.5, Nat.1)
    range_sum_one(polynomial_mul_term_coeff(quadratic_polynomial(a, b, c), quadratic_polynomial(d, e, g), Nat.5))
    range_sum(polynomial_mul_term_coeff(quadratic_polynomial(a, b, c), quadratic_polynomial(d, e, g), Nat.5), Nat.1) =
        polynomial_mul_term_coeff(quadratic_polynomial(a, b, c), quadratic_polynomial(d, e, g), Nat.5, Nat.0)
    quadratic_polynomial_support_bounded_by_three(d, e, g)
    polynomial_support_bounded_by(quadratic_polynomial(d, e, g), Nat.3)
    quadratic_polynomial_support_bounded_by_three(a, b, c)
    polynomial_support_bounded_by(quadratic_polynomial(a, b, c), Nat.3)
    ring_mul_term_coeff_at(quadratic_polynomial(a, b, c), quadratic_polynomial(d, e, g), Nat.5, Nat.0)
    polynomial_mul_term_coeff(quadratic_polynomial(a, b, c), quadratic_polynomial(d, e, g), Nat.5, Nat.0) =
        quadratic_polynomial(a, b, c).coeff(Nat.0) * quadratic_polynomial(d, e, g).coeff(Nat.5 - Nat.0)
    ring_nat_sub_eq_of_add(Nat.5, Nat.0, Nat.5)
    Nat.5 + Nat.0 = Nat.5
    Nat.5 - Nat.0 = Nat.5
    quadratic_polynomial_coeff_zero(a, b, c)
    quadratic_polynomial(a, b, c).coeff(Nat.0) = c
    ring_nat_three_le_five
    Nat.3 <= Nat.5
    lte_imp_not_lt(Nat.3, Nat.5)
    not Nat.5 < Nat.3
    polynomial_support_bounded_by_apply(quadratic_polynomial(d, e, g), Nat.3, Nat.5)
    quadratic_polynomial(d, e, g).coeff(Nat.5) = R.0
    polynomial_mul_term_coeff(quadratic_polynomial(a, b, c), quadratic_polynomial(d, e, g), Nat.5, Nat.0) = c * R.0
    c * R.0 = R.0
    polynomial_mul_term_coeff(quadratic_polynomial(a, b, c), quadratic_polynomial(d, e, g), Nat.5, Nat.0) = R.0
    ring_mul_term_coeff_at(quadratic_polynomial(a, b, c), quadratic_polynomial(d, e, g), Nat.5, Nat.1)
    polynomial_mul_term_coeff(quadratic_polynomial(a, b, c), quadratic_polynomial(d, e, g), Nat.5, Nat.1) =
        quadratic_polynomial(a, b, c).coeff(Nat.1) * quadratic_polynomial(d, e, g).coeff(Nat.5 - Nat.1)
    ring_nat_sub_eq_of_add(Nat.5, Nat.1, Nat.4)
    Nat.4 + Nat.1 = Nat.5
    Nat.5 - Nat.1 = Nat.4
    quadratic_polynomial_coeff_one(a, b, c)
    quadratic_polynomial(a, b, c).coeff(Nat.1) = b
    ring_nat_three_le_four
    Nat.3 <= Nat.4
    lte_imp_not_lt(Nat.3, Nat.4)
    not Nat.4 < Nat.3
    polynomial_support_bounded_by_apply(quadratic_polynomial(d, e, g), Nat.3, Nat.4)
    quadratic_polynomial(d, e, g).coeff(Nat.4) = R.0
    polynomial_mul_term_coeff(quadratic_polynomial(a, b, c), quadratic_polynomial(d, e, g), Nat.5, Nat.1) = b * R.0
    b * R.0 = R.0
    polynomial_mul_term_coeff(quadratic_polynomial(a, b, c), quadratic_polynomial(d, e, g), Nat.5, Nat.1) = R.0
    ring_mul_term_coeff_at(quadratic_polynomial(a, b, c), quadratic_polynomial(d, e, g), Nat.5, Nat.2)
    polynomial_mul_term_coeff(quadratic_polynomial(a, b, c), quadratic_polynomial(d, e, g), Nat.5, Nat.2) =
        quadratic_polynomial(a, b, c).coeff(Nat.2) * quadratic_polynomial(d, e, g).coeff(Nat.5 - Nat.2)
    ring_nat_sub_eq_of_add(Nat.5, Nat.2, Nat.3)
    Nat.3 + Nat.2 = Nat.5
    Nat.5 - Nat.2 = Nat.3
    quadratic_polynomial_coeff_two(a, b, c)
    quadratic_polynomial(a, b, c).coeff(Nat.2) = a
    lt_not_ref(Nat.3)
    not Nat.3 < Nat.3
    polynomial_support_bounded_by_apply(quadratic_polynomial(d, e, g), Nat.3, Nat.3)
    quadratic_polynomial(d, e, g).coeff(Nat.3) = R.0
    polynomial_mul_term_coeff(quadratic_polynomial(a, b, c), quadratic_polynomial(d, e, g), Nat.5, Nat.2) = a * R.0
    a * R.0 = R.0
    polynomial_mul_term_coeff(quadratic_polynomial(a, b, c), quadratic_polynomial(d, e, g), Nat.5, Nat.2) = R.0
    ring_mul_term_coeff_at(quadratic_polynomial(a, b, c), quadratic_polynomial(d, e, g), Nat.5, Nat.3)
    polynomial_mul_term_coeff(quadratic_polynomial(a, b, c), quadratic_polynomial(d, e, g), Nat.5, Nat.3) =
        quadratic_polynomial(a, b, c).coeff(Nat.3) * quadratic_polynomial(d, e, g).coeff(Nat.5 - Nat.3)
    ring_nat_sub_eq_of_add(Nat.5, Nat.3, Nat.2)
    Nat.2 + Nat.3 = Nat.5
    Nat.5 - Nat.3 = Nat.2
    polynomial_support_bounded_by_apply(quadratic_polynomial(a, b, c), Nat.3, Nat.3)
    quadratic_polynomial(a, b, c).coeff(Nat.3) = R.0
    quadratic_polynomial_coeff_two(d, e, g)
    quadratic_polynomial(d, e, g).coeff(Nat.2) = d
    polynomial_mul_term_coeff(quadratic_polynomial(a, b, c), quadratic_polynomial(d, e, g), Nat.5, Nat.3) = R.0 * d
    R.0 * d = R.0
    polynomial_mul_term_coeff(quadratic_polynomial(a, b, c), quadratic_polynomial(d, e, g), Nat.5, Nat.3) = R.0
    ring_mul_term_coeff_at(quadratic_polynomial(a, b, c), quadratic_polynomial(d, e, g), Nat.5, Nat.4)
    polynomial_mul_term_coeff(quadratic_polynomial(a, b, c), quadratic_polynomial(d, e, g), Nat.5, Nat.4) =
        quadratic_polynomial(a, b, c).coeff(Nat.4) * quadratic_polynomial(d, e, g).coeff(Nat.5 - Nat.4)
    ring_nat_sub_eq_of_add(Nat.5, Nat.4, Nat.1)
    Nat.1 + Nat.4 = Nat.5
    Nat.5 - Nat.4 = Nat.1
    ring_nat_three_le_four
    Nat.3 <= Nat.4
    lte_imp_not_lt(Nat.3, Nat.4)
    not Nat.4 < Nat.3
    polynomial_support_bounded_by_apply(quadratic_polynomial(a, b, c), Nat.3, Nat.4)
    quadratic_polynomial(a, b, c).coeff(Nat.4) = R.0
    quadratic_polynomial_coeff_one(d, e, g)
    quadratic_polynomial(d, e, g).coeff(Nat.1) = e
    polynomial_mul_term_coeff(quadratic_polynomial(a, b, c), quadratic_polynomial(d, e, g), Nat.5, Nat.4) = R.0 * e
    R.0 * e = R.0
    polynomial_mul_term_coeff(quadratic_polynomial(a, b, c), quadratic_polynomial(d, e, g), Nat.5, Nat.4) = R.0
    ring_mul_term_coeff_at(quadratic_polynomial(a, b, c), quadratic_polynomial(d, e, g), Nat.5, Nat.5)
    polynomial_mul_term_coeff(quadratic_polynomial(a, b, c), quadratic_polynomial(d, e, g), Nat.5, Nat.5) =
        quadratic_polynomial(a, b, c).coeff(Nat.5) * quadratic_polynomial(d, e, g).coeff(Nat.5 - Nat.5)
    ring_nat_sub_eq_of_add(Nat.5, Nat.5, Nat.0)
    Nat.0 + Nat.5 = Nat.5
    Nat.5 - Nat.5 = Nat.0
    ring_nat_three_le_five
    Nat.3 <= Nat.5
    lte_imp_not_lt(Nat.3, Nat.5)
    not Nat.5 < Nat.3
    polynomial_support_bounded_by_apply(quadratic_polynomial(a, b, c), Nat.3, Nat.5)
    quadratic_polynomial(a, b, c).coeff(Nat.5) = R.0
    quadratic_polynomial_coeff_zero(d, e, g)
    quadratic_polynomial(d, e, g).coeff(Nat.0) = g
    polynomial_mul_term_coeff(quadratic_polynomial(a, b, c), quadratic_polynomial(d, e, g), Nat.5, Nat.5) = R.0 * g
    R.0 * g = R.0
    polynomial_mul_term_coeff(quadratic_polynomial(a, b, c), quadratic_polynomial(d, e, g), Nat.5, Nat.5) = R.0
    range_sum(polynomial_mul_term_coeff(quadratic_polynomial(a, b, c), quadratic_polynomial(d, e, g), Nat.5), Nat.1) = R.0
    range_sum(polynomial_mul_term_coeff(quadratic_polynomial(a, b, c), quadratic_polynomial(d, e, g), Nat.5), Nat.2) = R.0 + R.0
    range_sum(polynomial_mul_term_coeff(quadratic_polynomial(a, b, c), quadratic_polynomial(d, e, g), Nat.5), Nat.3) = R.0 + R.0 + R.0
    range_sum(polynomial_mul_term_coeff(quadratic_polynomial(a, b, c), quadratic_polynomial(d, e, g), Nat.5), Nat.4) = R.0 + R.0 + R.0 + R.0
    range_sum(polynomial_mul_term_coeff(quadratic_polynomial(a, b, c), quadratic_polynomial(d, e, g), Nat.5), Nat.5) = R.0 + R.0 + R.0 + R.0 + R.0
    range_sum(polynomial_mul_term_coeff(quadratic_polynomial(a, b, c), quadratic_polynomial(d, e, g), Nat.5), Nat.6) = R.0 + R.0 + R.0 + R.0 + R.0 + R.0
    R.0 + R.0 + R.0 + R.0 + R.0 + R.0 = R.0
    polynomial_mul_coeff(quadratic_polynomial(a, b, c), quadratic_polynomial(d, e, g), Nat.5) = R.0
    polynomial_mul(quadratic_polynomial(a, b, c), quadratic_polynomial(d, e, g)).coeff(Nat.5) = R.0
}

/// The product of two quadratics has no nonzero coefficient from the fifth
/// slot on: the sharp bound for the product degree.
theorem quadratic_mul_support_bounded_by_five[R: CommRing](
    a: R, b: R, c: R, d: R, e: R, g: R
) {
    polynomial_support_bounded_by(polynomial_mul(quadratic_polynomial(a, b, c), quadratic_polynomial(d, e, g)), Nat.5)
} by {
    forall(k: Nat) {
        if k < Nat.5 {
            coeff_zero_from_at(polynomial_mul(quadratic_polynomial(a, b, c), quadratic_polynomial(d, e, g)).coeff, Nat.5, k) =
                (not k < Nat.5 implies coeff_zero_at(polynomial_mul(quadratic_polynomial(a, b, c), quadratic_polynomial(d, e, g)).coeff, k))
            coeff_zero_from_at(polynomial_mul(quadratic_polynomial(a, b, c), quadratic_polynomial(d, e, g)).coeff, Nat.5, k)
        } else {
            not k < Nat.5
            if k < Nat.6 {
                trichotomy(Nat.5, k)
                if Nat.5 < k {
                    lt_suc_right(k, Nat.5)
                    k = Nat.5 or k < Nat.5
                    false
                }
                not Nat.5 < k
                not k < Nat.5
                Nat.5 = k
                quadratic_mul_coeff_five(a, b, c, d, e, g)
                polynomial_mul(quadratic_polynomial(a, b, c), quadratic_polynomial(d, e, g)).coeff(Nat.5) = R.0
                polynomial_mul(quadratic_polynomial(a, b, c), quadratic_polynomial(d, e, g)).coeff(k) = R.0
                coeff_zero_at(polynomial_mul(quadratic_polynomial(a, b, c), quadratic_polynomial(d, e, g)).coeff, k)
            } else {
                not k < Nat.6
                polynomial_mul_coeff_apply(quadratic_polynomial(a, b, c), quadratic_polynomial(d, e, g), k)
                polynomial_mul(quadratic_polynomial(a, b, c), quadratic_polynomial(d, e, g)).coeff(k) =
                    polynomial_mul_coeff(quadratic_polynomial(a, b, c), quadratic_polynomial(d, e, g), k)
                quadratic_polynomial_support_bounded_by_three(a, b, c)
                polynomial_support_bounded_by(quadratic_polynomial(a, b, c), Nat.3)
                quadratic_polynomial_support_bounded_by_three(d, e, g)
                polynomial_support_bounded_by(quadratic_polynomial(d, e, g), Nat.3)
                polynomial_mul_coeff_zero_of_support_bounded(quadratic_polynomial(a, b, c), quadratic_polynomial(d, e, g), Nat.3, Nat.3, k)
                polynomial_mul_coeff(quadratic_polynomial(a, b, c), quadratic_polynomial(d, e, g), k) = R.0
                polynomial_mul(quadratic_polynomial(a, b, c), quadratic_polynomial(d, e, g)).coeff(k) = R.0
                coeff_zero_at(polynomial_mul(quadratic_polynomial(a, b, c), quadratic_polynomial(d, e, g)).coeff, k)
            }
            ring_coeff_zero_from_at_intro(polynomial_mul(quadratic_polynomial(a, b, c), quadratic_polynomial(d, e, g)).coeff, Nat.5, k)
            coeff_zero_from_at(polynomial_mul(quadratic_polynomial(a, b, c), quadratic_polynomial(d, e, g)).coeff, Nat.5, k)
        }
    }
    ring_coeff_zero_from_intro(polynomial_mul(quadratic_polynomial(a, b, c), quadratic_polynomial(d, e, g)).coeff, Nat.5)
    coeff_zero_from(polynomial_mul(quadratic_polynomial(a, b, c), quadratic_polynomial(d, e, g)).coeff, Nat.5)
    polynomial_support_bounded_by(polynomial_mul(quadratic_polynomial(a, b, c), quadratic_polynomial(d, e, g)), Nat.5) =
        coeff_zero_from(polynomial_mul(quadratic_polynomial(a, b, c), quadratic_polynomial(d, e, g)).coeff, Nat.5)
    polynomial_support_bounded_by(polynomial_mul(quadratic_polynomial(a, b, c), quadratic_polynomial(d, e, g)), Nat.5)
}

/// Over a field, the product of two quadratics with nonzero leading
/// coefficients has degree exactly four: `deg(f * g) = deg f + deg g` for
/// quadratics.
theorem quadratic_product_degree[F: Field](a: F, b: F, c: F, d: F, e: F, g: F) {
    a != F.0 and d != F.0 implies
    polynomial_degree_le(polynomial_mul(quadratic_polynomial(a, b, c), quadratic_polynomial(d, e, g)), Nat.4) and
    not polynomial_degree_le(polynomial_mul(quadratic_polynomial(a, b, c), quadratic_polynomial(d, e, g)), Nat.3)
} by {
    if a != F.0 and d != F.0 {
        quadratic_mul_support_bounded_by_five(a, b, c, d, e, g)
        polynomial_support_bounded_by(polynomial_mul(quadratic_polynomial(a, b, c), quadratic_polynomial(d, e, g)), Nat.5)
        polynomial_degree_le(polynomial_mul(quadratic_polynomial(a, b, c), quadratic_polynomial(d, e, g)), Nat.4) =
            polynomial_support_bounded_by(polynomial_mul(quadratic_polynomial(a, b, c), quadratic_polynomial(d, e, g)), Nat.4.suc)
        polynomial_degree_le(polynomial_mul(quadratic_polynomial(a, b, c), quadratic_polynomial(d, e, g)), Nat.4)
        if polynomial_support_bounded_by(polynomial_mul(quadratic_polynomial(a, b, c), quadratic_polynomial(d, e, g)), Nat.4) {
            lt_not_ref(Nat.4)
            not Nat.4 < Nat.4
            polynomial_support_bounded_by_apply(polynomial_mul(quadratic_polynomial(a, b, c), quadratic_polynomial(d, e, g)), Nat.4, Nat.4)
            polynomial_mul(quadratic_polynomial(a, b, c), quadratic_polynomial(d, e, g)).coeff(Nat.4) = F.0
            quadratic_mul_coeff_four(a, b, c, d, e, g)
            polynomial_mul(quadratic_polynomial(a, b, c), quadratic_polynomial(d, e, g)).coeff(Nat.4) = a * d
            a * d = F.0
            field_mul_eq_zero(a, d)
            if a = F.0 {
                false
            }
            d = F.0
            false
        }
        not polynomial_support_bounded_by(polynomial_mul(quadratic_polynomial(a, b, c), quadratic_polynomial(d, e, g)), Nat.4)
        polynomial_degree_le(polynomial_mul(quadratic_polynomial(a, b, c), quadratic_polynomial(d, e, g)), Nat.3) =
            polynomial_support_bounded_by(polynomial_mul(quadratic_polynomial(a, b, c), quadratic_polynomial(d, e, g)), Nat.3.suc)
        not polynomial_degree_le(polynomial_mul(quadratic_polynomial(a, b, c), quadratic_polynomial(d, e, g)), Nat.3)
        polynomial_degree_le(polynomial_mul(quadratic_polynomial(a, b, c), quadratic_polynomial(d, e, g)), Nat.4) and
            not polynomial_degree_le(polynomial_mul(quadratic_polynomial(a, b, c), quadratic_polynomial(d, e, g)), Nat.3)
    }
}

/// The classical product-degree identity over a field: when the leading
/// coefficients of both factors are nonzero, `deg(f * g) = deg f + deg g`.
///
/// In the support-bound formulation: `f` supported below `m.suc` with a
/// nonzero coefficient at `m`, and `g` supported below `n.suc` with a
/// nonzero coefficient at `n`, force the product to be supported below
/// `m.suc + n` with a nonzero coefficient at `m + n`.  The sharp bound needs
/// the full convolution argument one slot past the generic bound
/// (`polynomial_mul_support_bounded_by_add`), which the library does not
/// provide in that generality; the quadratic case is proved above.
// theorem polynomial_degree_mul_eq_add[F: Field](
//     p: Polynomial[F], q: Polynomial[F], m: Nat, n: Nat
// ) {
//     polynomial_support_bounded_by(p, m.suc) and p.coeff(m) != F.0 and
//     polynomial_support_bounded_by(q, n.suc) and q.coeff(n) != F.0 implies
//     polynomial_support_bounded_by(polynomial_mul(p, q), m.suc + n) and
//     polynomial_mul(p, q).coeff(m + n) != F.0
// }

// ---------------------------------------------------------------------------
// Evaluation as a ring homomorphism
// ---------------------------------------------------------------------------

/// Evaluation at a point preserves addition and multiplication: the
/// evaluation homomorphism.
theorem polynomial_eval_homomorphism[R: CommRing](x: R, p: Polynomial[R], q: Polynomial[R]) {
    polynomial_eval(p + q, x) = polynomial_eval(p, x) + polynomial_eval(q, x) and
    polynomial_eval(polynomial_mul(p, q), x) = polynomial_eval(p, x) * polynomial_eval(q, x)
} by {
    polynomial_eval_add(p, q, x)
    polynomial_eval(p + q, x) = polynomial_eval(p, x) + polynomial_eval(q, x)
    polynomial_eval_mul(p, q, x)
    polynomial_eval(polynomial_mul(p, q), x) = polynomial_eval(p, x) * polynomial_eval(q, x)
}

/// The polynomial `X - 1` over a field.
let field_x_minus_one[F: Field]: Polynomial[F] = linear_polynomial(F.1, -F.1)

/// The polynomial `X + 1` over a field.
let field_x_plus_one[F: Field]: Polynomial[F] = linear_polynomial(F.1, F.1)

/// The polynomial `X^2 - 1` over a field.
let field_x_squared_minus_one[F: Field]: Polynomial[F] = quadratic_polynomial(F.1, F.0, -F.1)

// ---------------------------------------------------------------------------
// Ring arithmetic used by the concrete evaluations
// ---------------------------------------------------------------------------

/// `(x - 1) + (x + 1) = 2x` in coefficient form.
lemma ring_add_x_minus_one_x_plus_one[R: CommRing](x: R) {
    (-R.1 + R.1 * x) + (R.1 + R.1 * x) = (R.1 + R.1) * x
} by {
    (-R.1 + R.1 * x) + (R.1 + R.1 * x) = (R.1 * x + -R.1) + (R.1 + R.1 * x)
    (R.1 * x + -R.1) + (R.1 + R.1 * x) = R.1 * x + (-R.1 + (R.1 + R.1 * x))
    R.1 * x + (-R.1 + (R.1 + R.1 * x)) = R.1 * x + ((-R.1 + R.1) + R.1 * x)
    inverse_left(R.1)
    -R.1 + R.1 = R.0
    R.1 * x + ((-R.1 + R.1) + R.1 * x) = R.1 * x + (R.0 + R.1 * x)
    R.1 * x + (R.0 + R.1 * x) = R.1 * x + R.1 * x
    R.1 * x + R.1 * x = (R.1 + R.1) * x
}

/// `(x - 1)(x + 1) = x^2 - 1`.
lemma ring_mul_x_minus_one_x_plus_one[R: CommRing](x: R) {
    (x - R.1) * (x + R.1) = x * x - R.1
} by {
    (x - R.1) * (x + R.1) = x * (x + R.1) - R.1 * (x + R.1)
    x * (x + R.1) = x * x + x * R.1
    x * R.1 = x
    x * (x + R.1) = x * x + x
    R.1 * (x + R.1) = R.1 * x + R.1 * R.1
    R.1 * x = x
    R.1 * R.1 = R.1
    R.1 * (x + R.1) = x + R.1
    (x - R.1) * (x + R.1) = (x * x + x) - (x + R.1)
    (x * x + x) - (x + R.1) = (x * x + x) + -(x + R.1)
    inverse_add(x, R.1)
    -(x + R.1) = -R.1 + -x
    (x * x + x) + -(x + R.1) = (x * x + x) + (-R.1 + -x)
    (x * x + x) + (-R.1 + -x) = x * x + (x + (-R.1 + -x))
    x * x + (x + (-R.1 + -x)) = x * x + ((x + -R.1) + -x)
    x * x + ((x + -R.1) + -x) = x * x + ((-R.1 + x) + -x)
    x * x + ((-R.1 + x) + -x) = x * x + (-R.1 + (x + -x))
    x + -x = R.0
    x * x + (-R.1 + (x + -x)) = x * x + (-R.1 + R.0)
    x * x + (-R.1 + R.0) = x * x + -R.1
    x * x + -R.1 = x * x - R.1
    (x * x + x) - (x + R.1) = x * x - R.1
    (x - R.1) * (x + R.1) = x * x - R.1
}

/// Concretely, evaluation preserves addition for the polynomials `X - 1` and
/// `X + 1`: `(X - 1 + X + 1)(x) = (1 + 1) * x`.
theorem field_eval_add_concrete[F: Field](x: F) {
    polynomial_eval(field_x_minus_one[F] + field_x_plus_one[F], x) = (F.1 + F.1) * x
} by {
    polynomial_eval_add(field_x_minus_one[F], field_x_plus_one[F], x)
    polynomial_eval(field_x_minus_one[F] + field_x_plus_one[F], x) =
        polynomial_eval(field_x_minus_one[F], x) + polynomial_eval(field_x_plus_one[F], x)
    linear_polynomial_eval(F.1, -F.1, x)
    polynomial_eval(field_x_minus_one[F], x) = -F.1 + F.1 * x
    linear_polynomial_eval(F.1, F.1, x)
    polynomial_eval(field_x_plus_one[F], x) = F.1 + F.1 * x
    polynomial_eval(field_x_minus_one[F] + field_x_plus_one[F], x) = (-F.1 + F.1 * x) + (F.1 + F.1 * x)
    ring_add_x_minus_one_x_plus_one(x)
    (-F.1 + F.1 * x) + (F.1 + F.1 * x) = (F.1 + F.1) * x
    polynomial_eval(field_x_minus_one[F] + field_x_plus_one[F], x) = (F.1 + F.1) * x
}

/// Concretely, evaluation preserves multiplication for the polynomials
/// `X - 1` and `X + 1`: `((X - 1)(X + 1))(x) = x^2 - 1`.
theorem field_eval_mul_concrete[F: Field](x: F) {
    polynomial_eval(polynomial_mul(field_x_minus_one[F], field_x_plus_one[F]), x) = x * x - F.1
} by {
    polynomial_eval_mul(field_x_minus_one[F], field_x_plus_one[F], x)
    polynomial_eval(polynomial_mul(field_x_minus_one[F], field_x_plus_one[F]), x) =
        polynomial_eval(field_x_minus_one[F], x) * polynomial_eval(field_x_plus_one[F], x)
    linear_polynomial_eval(F.1, -F.1, x)
    polynomial_eval(field_x_minus_one[F], x) = -F.1 + F.1 * x
    linear_polynomial_eval(F.1, F.1, x)
    polynomial_eval(field_x_plus_one[F], x) = F.1 + F.1 * x
    polynomial_eval(polynomial_mul(field_x_minus_one[F], field_x_plus_one[F]), x) =
        (-F.1 + F.1 * x) * (F.1 + F.1 * x)
    (-F.1 + F.1 * x) * (F.1 + F.1 * x) = (x - F.1) * (x + F.1)
    ring_mul_x_minus_one_x_plus_one(x)
    (x - F.1) * (x + F.1) = x * x - F.1
    (-F.1 + F.1 * x) * (F.1 + F.1 * x) = x * x - F.1
    polynomial_eval(polynomial_mul(field_x_minus_one[F], field_x_plus_one[F]), x) = x * x - F.1
}

// ---------------------------------------------------------------------------
// The remainder theorem for quadratics
// ---------------------------------------------------------------------------

/// The remainder theorem: `f(a) = 0` if and only if `X - a` divides `f`.
theorem polynomial_remainder_theorem_global[R: CommRing](p: Polynomial[R], a: R) {
    polynomial_eval(p, a) = R.0 iff polynomial_divisible_by_x_minus_a_polynomial(p, a)
} by {
    polynomial_factor_theorem_global(p, a)
}

/// The concrete quadratic `X^2 - 1` has `1` as a root.
theorem field_x_squared_minus_one_has_root_one[F: Field] {
    polynomial_eval(field_x_squared_minus_one[F], F.1) = F.0
} by {
    quadratic_polynomial_eval(F.1, F.0, -F.1, F.1)
    polynomial_eval(field_x_squared_minus_one[F], F.1) = -F.1 + F.0 * F.1 + F.1 * F.1 * F.1
    F.0 * F.1 = F.0
    F.1 * F.1 * F.1 = F.1
    -F.1 + F.0 + F.1 * F.1 * F.1 = -F.1 + F.0 + F.1
    -F.1 + F.0 + F.1 = -F.1 + F.1
    inverse_left(F.1)
    -F.1 + F.1 = F.0
    -F.1 + F.0 + F.1 * F.1 * F.1 = F.0
    polynomial_eval(field_x_squared_minus_one[F], F.1) = F.0
}

/// `X - 1` divides `X^2 - 1`: the forward direction of the remainder theorem
/// for the concrete quadratic.
theorem field_x_minus_one_divides_x_squared_minus_one[F: Field] {
    polynomial_divisible_by_x_minus_a_polynomial(field_x_squared_minus_one[F], F.1)
} by {
    field_x_squared_minus_one_has_root_one[F]
    polynomial_eval(field_x_squared_minus_one[F], F.1) = F.0
    polynomial_factor_theorem_forward_global(field_x_squared_minus_one[F], F.1)
    polynomial_divisible_by_x_minus_a_polynomial(field_x_squared_minus_one[F], F.1)
}

/// Divisibility by `X - 1` forces `1` to be a root of `X^2 - 1`: the reverse
/// direction of the remainder theorem for the concrete quadratic.
theorem field_x_minus_one_divides_imp_root_one[F: Field] {
    polynomial_divisible_by_x_minus_a_polynomial(field_x_squared_minus_one[F], F.1) implies
    polynomial_eval(field_x_squared_minus_one[F], F.1) = F.0
} by {
    polynomial_factor_theorem_reverse_global(field_x_squared_minus_one[F], F.1)
}

/// The remainder theorem for the concrete quadratic: `1` is a root of
/// `X^2 - 1` exactly when `X - 1` divides it.
theorem field_x_squared_minus_one_root_iff_divisible[F: Field] {
    polynomial_eval(field_x_squared_minus_one[F], F.1) = F.0 iff
        polynomial_divisible_by_x_minus_a_polynomial(field_x_squared_minus_one[F], F.1)
} by {
    polynomial_factor_theorem_global(field_x_squared_minus_one[F], F.1)
}

/// The explicit quotient: `X + 1` witnesses the divisibility, so
/// `(X^2 - 1)(x) = (X + 1)(x) * (x - 1)` at every point `x`.
theorem field_x_plus_one_witnesses_factor[F: Field] {
    polynomial_x_minus_a_polynomial_witness(field_x_squared_minus_one[F], F.1, field_x_plus_one[F])
} by {
    forall(x: F) {
        quadratic_polynomial_eval(F.1, F.0, -F.1, x)
        polynomial_eval(field_x_squared_minus_one[F], x) = -F.1 + F.0 * x + F.1 * x * x
        linear_polynomial_eval(F.1, F.1, x)
        polynomial_eval(field_x_plus_one[F], x) = F.1 + F.1 * x
        F.0 * x = F.0
        -F.1 + F.0 * x + F.1 * x * x = -F.1 + F.1 * x * x
        F.1 * x * x = x * x
        -F.1 + F.1 * x * x = -F.1 + x * x
        -F.1 + x * x = x * x - F.1
        (F.1 + F.1 * x) * (x - F.1) = (F.1 * x + F.1) * (x - F.1)
        (F.1 * x + F.1) * (x - F.1) = (x + F.1) * (x - F.1)
        (x + F.1) * (x - F.1) = (x - F.1) * (x + F.1)
        ring_mul_x_minus_one_x_plus_one(x)
        (x - F.1) * (x + F.1) = x * x - F.1
        (F.1 + F.1 * x) * (x - F.1) = x * x - F.1
        -F.1 + F.0 * x + F.1 * x * x = (F.1 + F.1 * x) * (x - F.1)
        polynomial_eval(field_x_squared_minus_one[F], x) =
            polynomial_eval(field_x_plus_one[F], x) * (x - F.1)
    }
    polynomial_x_minus_a_polynomial_witness(field_x_squared_minus_one[F], F.1, field_x_plus_one[F]) =
        forall(x: F) {
            polynomial_x_minus_a_polynomial_witness_at(field_x_squared_minus_one[F], F.1, field_x_plus_one[F], x)
        }
    polynomial_x_minus_a_polynomial_witness(field_x_squared_minus_one[F], F.1, field_x_plus_one[F])
}

// ---------------------------------------------------------------------------
// The integral domain property over a field
// ---------------------------------------------------------------------------

/// Polynomials over a field form an integral domain: a zero product has a
/// zero factor.
///
/// The general statement needs the degree machinery of the sharp product
/// bound; the degree-one case is proved below.
// theorem polynomial_field_integral_domain[F: Field](p: Polynomial[F], q: Polynomial[F]) {
//     polynomial_mul(p, q) = Polynomial[F].zero implies
//     p = Polynomial[F].zero or q = Polynomial[F].zero
// }

/// For degree-one polynomials over a field, a zero product forces the second
/// factor to be zero.
theorem linear_zero_product[F: Field](a: F, b: F, c: F, d: F) {
    a != F.0 and
    polynomial_mul(linear_polynomial(a, b), linear_polynomial(c, d)) = Polynomial[F].zero
    implies linear_polynomial(c, d) = Polynomial[F].zero
} by {
    if a != F.0 and
        polynomial_mul(linear_polynomial(a, b), linear_polynomial(c, d)) = Polynomial[F].zero {
        polynomial_eq_coeff_at(polynomial_mul(linear_polynomial(a, b), linear_polynomial(c, d)),
            Polynomial[F].zero, Nat.2)
        polynomial_mul(linear_polynomial(a, b), linear_polynomial(c, d)).coeff(Nat.2) = F.0
        linear_mul_coeff_two(a, b, c, d)
        polynomial_mul(linear_polynomial(a, b), linear_polynomial(c, d)).coeff(Nat.2) = a * c
        a * c = F.0
        field_mul_eq_zero(a, c)
        if a = F.0 {
            false
        }
        c = F.0
        polynomial_eq_coeff_at(polynomial_mul(linear_polynomial(a, b), linear_polynomial(c, d)),
            Polynomial[F].zero, Nat.1)
        polynomial_mul(linear_polynomial(a, b), linear_polynomial(c, d)).coeff(Nat.1) = F.0
        linear_mul_coeff_one(a, b, c, d)
        polynomial_mul(linear_polynomial(a, b), linear_polynomial(c, d)).coeff(Nat.1) = a * d + b * c
        a * d + b * c = F.0
        b * c = b * F.0
        b * F.0 = F.0
        a * d + b * c = a * d + F.0
        a * d + F.0 = a * d
        a * d = F.0
        field_mul_eq_zero(a, d)
        if a = F.0 {
            false
        }
        d = F.0
        forall(k: Nat) {
            if k = Nat.0 {
                linear_polynomial_coeff_zero(c, d)
                linear_polynomial(c, d).coeff(Nat.0) = d
                d = F.0
                linear_polynomial(c, d).coeff(k) = F.0
            } else {
                zero_or_suc(k)
                let kp: Nat satisfy {
                    k = kp.suc
                }
                k = kp.suc
                if kp < Nat.1 {
                    lt_suc_right(kp, Nat.0)
                    if kp < Nat.0 {
                        not_lt_zero(kp)
                        false
                    }
                    kp = Nat.0
                    k = kp.suc
                    k = Nat.1
                    linear_polynomial_coeff_one(c, d)
                    linear_polynomial(c, d).coeff(Nat.1) = c
                    c = F.0
                    linear_polynomial(c, d).coeff(k) = F.0
                } else {
                    not kp < Nat.1
                    linear_polynomial_support_bounded_by_two(c, d)
                    polynomial_support_bounded_by(linear_polynomial(c, d), Nat.2)
                    lt_or_lte(kp, Nat.1)
                    Nat.1 <= kp
                    lte_suc_suc(Nat.1, kp)
                    Nat.2 <= kp.suc
                    k = kp.suc
                    Nat.2 <= k
                    lte_imp_not_lt(Nat.2, k)
                    not k < Nat.2
                    polynomial_support_bounded_by_apply(linear_polynomial(c, d), Nat.2, k)
                    linear_polynomial(c, d).coeff(k) = F.0
                }
            }
        }
        polynomial_eq_zero_of_coeff_zero(linear_polynomial(c, d))
        linear_polynomial(c, d) = Polynomial[F].zero
    }
}

/// The factorization `X^2 - 1 = (X - 1)(X + 1)` as polynomials over a field.
theorem field_x_squared_minus_one_eq_mul[F: Field] {
    polynomial_mul(field_x_minus_one[F], field_x_plus_one[F]) = field_x_squared_minus_one[F]
} by {
    forall(k: Nat) {
        if k = Nat.0 {
            linear_mul_coeff_zero(F.1, -F.1, F.1, F.1)
            polynomial_mul(field_x_minus_one[F], field_x_plus_one[F]).coeff(Nat.0) = (-F.1) * F.1
            mul_neg_left(F.1, F.1)
            -F.1 * F.1 = -(F.1 * F.1)
            F.1 * F.1 = F.1
            -F.1 * F.1 = -F.1
            polynomial_mul(field_x_minus_one[F], field_x_plus_one[F]).coeff(Nat.0) = -F.1
            quadratic_polynomial_coeff_zero(F.1, F.0, -F.1)
            field_x_squared_minus_one[F].coeff(Nat.0) = -F.1
            polynomial_mul(field_x_minus_one[F], field_x_plus_one[F]).coeff(k) =
                field_x_squared_minus_one[F].coeff(k)
        } else {
            zero_or_suc(k)
            let kp: Nat satisfy {
                k = kp.suc
            }
            k = kp.suc
            if kp = Nat.0 {
                k = Nat.1
                linear_mul_coeff_one(F.1, -F.1, F.1, F.1)
                polynomial_mul(field_x_minus_one[F], field_x_plus_one[F]).coeff(Nat.1) =
                    F.1 * F.1 + (-F.1) * F.1
                mul_neg_left(F.1, F.1)
                -F.1 * F.1 = -(F.1 * F.1)
                F.1 * F.1 = F.1
                -F.1 * F.1 = -F.1
                F.1 * F.1 + (-F.1) * F.1 = F.1 + -F.1
                inverse_left(F.1)
                -F.1 + F.1 = F.0
                F.1 + -F.1 = F.0
                F.1 * F.1 + (-F.1) * F.1 = F.0
                polynomial_mul(field_x_minus_one[F], field_x_plus_one[F]).coeff(Nat.1) = F.0
                quadratic_polynomial_coeff_one(F.1, F.0, -F.1)
                field_x_squared_minus_one[F].coeff(Nat.1) = F.0
                polynomial_mul(field_x_minus_one[F], field_x_plus_one[F]).coeff(k) =
                    field_x_squared_minus_one[F].coeff(k)
            } else {
                zero_or_suc(kp)
                let kq: Nat satisfy {
                    kp = kq.suc
                }
                kp = kq.suc
                if kq = Nat.0 {
                    k = Nat.2
                    linear_mul_coeff_two(F.1, -F.1, F.1, F.1)
                    polynomial_mul(field_x_minus_one[F], field_x_plus_one[F]).coeff(Nat.2) =
                        F.1 * F.1
                    F.1 * F.1 = F.1
                    polynomial_mul(field_x_minus_one[F], field_x_plus_one[F]).coeff(Nat.2) = F.1
                    quadratic_polynomial_coeff_two(F.1, F.0, -F.1)
                    field_x_squared_minus_one[F].coeff(Nat.2) = F.1
                    polynomial_mul(field_x_minus_one[F], field_x_plus_one[F]).coeff(k) =
                        field_x_squared_minus_one[F].coeff(k)
                } else {
                    zero_or_suc(kq)
                    let kr: Nat satisfy {
                        kq = kr.suc
                    }
                    kq = kr.suc
                    if kr < Nat.1 {
                        lt_suc_right(kr, Nat.0)
                        if kr < Nat.0 {
                            not_lt_zero(kr)
                            false
                        }
                        kr = Nat.0
                        kq = kr.suc
                        kp = kq.suc
                        k = kp.suc
                        k = Nat.3
                        linear_mul_coeff_three(F.1, -F.1, F.1, F.1)
                        polynomial_mul(field_x_minus_one[F], field_x_plus_one[F]).coeff(Nat.3) = F.0
                        quadratic_polynomial_support_bounded_by_three(F.1, F.0, -F.1)
                        polynomial_support_bounded_by(field_x_squared_minus_one[F], Nat.3)
                        lt_not_ref(Nat.3)
                        not Nat.3 < Nat.3
                        polynomial_support_bounded_by_apply(field_x_squared_minus_one[F], Nat.3, Nat.3)
                        field_x_squared_minus_one[F].coeff(Nat.3) = F.0
                        polynomial_mul(field_x_minus_one[F], field_x_plus_one[F]).coeff(k) =
                            field_x_squared_minus_one[F].coeff(k)
                    } else {
                        not kr < Nat.1
                        linear_mul_support_bounded_by_four(F.1, -F.1, F.1, F.1)
                        polynomial_support_bounded_by(polynomial_mul(field_x_minus_one[F], field_x_plus_one[F]), Nat.4)
                        lt_or_lte(kr, Nat.1)
                        Nat.1 <= kr
                        lte_suc_suc(Nat.1, kr)
                        Nat.2 <= kr.suc
                        kq = kr.suc
                        Nat.2 <= kq
                        lte_suc_suc(Nat.2, kq)
                        Nat.3 <= kq.suc
                        kp = kq.suc
                        Nat.3 <= kp
                        lte_suc_suc(Nat.3, kp)
                        Nat.4 <= kp.suc
                        k = kp.suc
                        Nat.4 <= k
                        lte_imp_not_lt(Nat.4, k)
                        not k < Nat.4
                        polynomial_support_bounded_by_apply(
                            polynomial_mul(field_x_minus_one[F], field_x_plus_one[F]), Nat.4, k)
                        polynomial_mul(field_x_minus_one[F], field_x_plus_one[F]).coeff(k) = F.0
                        quadratic_polynomial_support_bounded_by_three(F.1, F.0, -F.1)
                        polynomial_support_bounded_by(field_x_squared_minus_one[F], Nat.3)
                        ring_nat_three_le_four
                        Nat.3 <= Nat.4
                        lte_trans(Nat.3, Nat.4, k)
                        Nat.3 <= k
                        lte_imp_not_lt(Nat.3, k)
                        not k < Nat.3
                        polynomial_support_bounded_by_apply(field_x_squared_minus_one[F], Nat.3, k)
                        field_x_squared_minus_one[F].coeff(k) = F.0
                        polynomial_mul(field_x_minus_one[F], field_x_plus_one[F]).coeff(k) =
                            field_x_squared_minus_one[F].coeff(k)
                    }
                }
            }
        }
    }
    polynomial_ext_pointwise(polynomial_mul(field_x_minus_one[F], field_x_plus_one[F]),
        field_x_squared_minus_one[F])
    polynomial_mul(field_x_minus_one[F], field_x_plus_one[F]) = field_x_squared_minus_one[F]
}
