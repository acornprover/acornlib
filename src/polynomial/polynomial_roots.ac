/// Roots of polynomials: roots of constant, linear and monomial
/// polynomials, the zero-product property for roots, and how roots behave
/// under translation and scaling of the evaluation point.

from nat import Nat
from semiring import Semiring
from comm_ring import CommRing
from algebra.field.field import Field, field_mul_eq_zero, mul_inverse_right
from algebra.ring.ring import mul_neg_right
from algebra.add_group import inverse_add, left_cancel
from algebra.semigroup import Semigroup
from algebra.comm_semigroup import CommSemigroup
from polynomial import Polynomial, polynomial_constant, polynomial_monomial,
    polynomial_ext_pointwise, polynomial_eval, polynomial_eval_mul, polynomial_mul
from polynomial.polynomial_eval_deep import polynomial_eval_monomial, polynomial_eval_zero_point,
    polynomial_eval_constant_eq_zero
from polynomial.polynomial_vieta import linear_polynomial, linear_polynomial_eval,
    linear_polynomial_neg_root

numerals Nat

/// The value of `p` after translating the evaluation point by `a`.
define polynomial_shift_eval[R: CommRing](p: Polynomial[R], a: R, x: R) -> R {
    polynomial_eval(p, x + a)
}

/// The value of `p` after scaling the evaluation point by `c`.
define polynomial_scale_eval[R: CommRing](p: Polynomial[R], c: R, x: R) -> R {
    polynomial_eval(p, c * x)
}

/// `(x - a) + a = x` in a commutative ring.
lemma roots_add_sub_cancel[R: CommRing](a: R, x: R) {
    (x - a) + a = x
} by {
    x - a = x + -a
    (x - a) + a = (x + -a) + a
    (x + -a) + a = x + (-a + a)
    -a + a = a + -a
    a + -a = R.0
    x + (-a + a) = x + R.0
    x + R.0 = x
    (x - a) + a = x
}

/// The first power of an element is itself.
lemma roots_pow_one[R: Semiring](x: R) {
    x.pow(Nat.1) = x
} by {
    x.pow(Nat.1) = x.pow(Nat.0.suc)
    x.pow(Nat.0.suc) = x * x.pow(Nat.0)
    x.pow(Nat.0) = R.1
    x.pow(Nat.1) = x * R.1
    x * R.1 = x
}

/// A difference is zero exactly when its terms are equal.
theorem polynomial_sub_eq_zero_iff[R: CommRing](x: R, a: R) {
    (x - a = R.0) = (x = a)
} by {
    if x - a = R.0 {
        (x - a) + a = R.0 + a
        roots_add_sub_cancel(a, x)
        (x - a) + a = x
        R.0 + a = a
        x = a
    }
    if x = a {
        x - a = a - a
        a - a = a + -a
        a + -a = R.0
        x - a = R.0
    }
}

/// The polynomial `X - a` has exactly `a` as its root.
theorem polynomial_root_linear_one[R: CommRing](a: R, x: R) {
    (polynomial_eval(linear_polynomial(R.1, -a), x) = R.0) = (x = a)
} by {
    linear_polynomial_eval(R.1, -a, x)
    polynomial_eval(linear_polynomial(R.1, -a), x) = -a + R.1 * x
    R.1 * x = x
    polynomial_eval(linear_polynomial(R.1, -a), x) = -a + x
    -a + x = x + -a
    x + -a = x - a
    polynomial_eval(linear_polynomial(R.1, -a), x) = x - a
    polynomial_sub_eq_zero_iff(x, a)
    (x - a = R.0) = (x = a)
    (polynomial_eval(linear_polynomial(R.1, -a), x) = R.0) = (x = a)
}

/// A general linear polynomial vanishes exactly at the solutions of
/// `b + a * x = 0`.
theorem polynomial_root_linear[R: CommRing](a: R, b: R, x: R) {
    (polynomial_eval(linear_polynomial(a, b), x) = R.0) = (b + a * x = R.0)
} by {
    linear_polynomial_eval(a, b, x)
    polynomial_eval(linear_polynomial(a, b), x) = b + a * x
    (polynomial_eval(linear_polynomial(a, b), x) = R.0) = (b + a * x = R.0)
}

/// Over a field, a nondegenerate linear polynomial has a unique root.
theorem polynomial_linear_root_unique[F: Field](a: F, b: F, x: F, y: F) {
    a != F.0 and polynomial_eval(linear_polynomial(a, b), x) = F.0 and
    polynomial_eval(linear_polynomial(a, b), y) = F.0 implies x = y
} by {
    if a != F.0 and polynomial_eval(linear_polynomial(a, b), x) = F.0 and
        polynomial_eval(linear_polynomial(a, b), y) = F.0 {
        polynomial_root_linear(a, b, x)
        (polynomial_eval(linear_polynomial(a, b), x) = F.0) = (b + a * x = F.0)
        b + a * x = F.0
        polynomial_root_linear(a, b, y)
        (polynomial_eval(linear_polynomial(a, b), y) = F.0) = (b + a * y = F.0)
        b + a * y = F.0
        left_cancel(b, a * x, a * y)
        a * x = a * y
        a * x + -(a * y) = a * y + -(a * y)
        a * y + -(a * y) = F.0
        a * x + -(a * y) = F.0
        x - y = x + -y
        a * (x - y) = a * (x + -y)
        a * (x + -y) = a * x + a * -y
        mul_neg_right(a, y)
        a * -y = -(a * y)
        a * x + a * -y = a * x + -(a * y)
        a * (x - y) = a * x + -(a * y)
        a * (x - y) = F.0
        field_mul_eq_zero(a, x - y)
        if a = F.0 {
            false
        }
        x - y = F.0
        polynomial_sub_eq_zero_iff(x, y)
        (x - y = F.0) = (x = y)
        x = y
    }
}

/// The explicit root of a nondegenerate linear polynomial over a field.
theorem polynomial_linear_root_inverse[F: Field](a: F, b: F) {
    a != F.0 implies polynomial_eval(linear_polynomial(a, b), -b * a.inverse) = F.0
} by {
    if a != F.0 {
        linear_polynomial_eval(a, b, -b * a.inverse)
        polynomial_eval(linear_polynomial(a, b), -b * a.inverse) = b + a * (-b * a.inverse)
        Semigroup.mul_associative[F](a, -b, a.inverse)
        a * (-b * a.inverse) = (a * -b) * a.inverse
        a * -b = -b * a
        (a * -b) * a.inverse = (-b * a) * a.inverse
        Semigroup.mul_associative[F](-b, a, a.inverse)
        -b * (a * a.inverse) = (-b * a) * a.inverse
        mul_inverse_right(a)
        a * a.inverse = F.1
        -b * (a * a.inverse) = -b * F.1
        -b * F.1 = -b
        a * (-b * a.inverse) = -b
        polynomial_eval(linear_polynomial(a, b), -b * a.inverse) = b + -b
        b + -b = F.0
        polynomial_eval(linear_polynomial(a, b), -b * a.inverse) = F.0
    }
}

/// Over a field, every nondegenerate linear polynomial has a root.
theorem polynomial_linear_has_root[F: Field](a: F, b: F) {
    a != F.0 implies exists(x: F) {
        polynomial_eval(linear_polynomial(a, b), x) = F.0
    }
} by {
    if a != F.0 {
        polynomial_linear_root_inverse(a, b)
        polynomial_eval(linear_polynomial(a, b), -b * a.inverse) = F.0
        exists(x: F) {
            x = -b * a.inverse and polynomial_eval(linear_polynomial(a, b), x) = F.0
        }
        exists(x: F) {
            polynomial_eval(linear_polynomial(a, b), x) = F.0
        }
    }
}

/// A constant polynomial has a root exactly when its coefficient is zero.
theorem polynomial_root_constant[R: Semiring](r: R, x: R) {
    (polynomial_eval(polynomial_constant(r), x) = R.0) = (r = R.0)
} by {
    polynomial_eval_constant_eq_zero(r, x)
}

/// A monomial of degree one has a root exactly when its coefficient or the
/// point is zero.
theorem polynomial_root_monomial_one[F: Field](r: F, x: F) {
    (polynomial_eval(polynomial_monomial(Nat.1, r), x) = F.0) = (r = F.0 or x = F.0)
} by {
    polynomial_eval_monomial(Nat.1, r, x)
    polynomial_eval(polynomial_monomial(Nat.1, r), x) = r * x.pow(Nat.1)
    roots_pow_one(x)
    polynomial_eval(polynomial_monomial(Nat.1, r), x) = r * x
    if polynomial_eval(polynomial_monomial(Nat.1, r), x) = F.0 {
        r * x = F.0
        field_mul_eq_zero(r, x)
        r = F.0 or x = F.0
    }
    if r = F.0 or x = F.0 {
        if r = F.0 {
            r * x = F.0 * x
            F.0 * x = F.0
            r * x = F.0
        } else {
            x = F.0
            r * x = r * F.0
            r * F.0 = F.0
            r * x = F.0
        }
        polynomial_eval(polynomial_monomial(Nat.1, r), x) = F.0
    }
}

/// Over a field, a root of a product is a root of one of the factors, and
/// conversely.
theorem polynomial_root_product_iff[F: Field](p: Polynomial[F], q: Polynomial[F], x: F) {
    (polynomial_eval(polynomial_mul(p, q), x) = F.0) =
        (polynomial_eval(p, x) = F.0 or polynomial_eval(q, x) = F.0)
} by {
    polynomial_eval_mul(p, q, x)
    polynomial_eval(polynomial_mul(p, q), x) = polynomial_eval(p, x) * polynomial_eval(q, x)
    if polynomial_eval(polynomial_mul(p, q), x) = F.0 {
        polynomial_eval(p, x) * polynomial_eval(q, x) = F.0
        field_mul_eq_zero(polynomial_eval(p, x), polynomial_eval(q, x))
        polynomial_eval(p, x) = F.0 or polynomial_eval(q, x) = F.0
    }
    if polynomial_eval(p, x) = F.0 or polynomial_eval(q, x) = F.0 {
        if polynomial_eval(p, x) = F.0 {
            polynomial_eval(p, x) * polynomial_eval(q, x) = F.0 * polynomial_eval(q, x)
            F.0 * polynomial_eval(q, x) = F.0
            polynomial_eval(p, x) * polynomial_eval(q, x) = F.0
        } else {
            polynomial_eval(q, x) = F.0
            polynomial_eval(p, x) * polynomial_eval(q, x) = polynomial_eval(p, x) * F.0
            polynomial_eval(p, x) * F.0 = F.0
            polynomial_eval(p, x) * polynomial_eval(q, x) = F.0
        }
        polynomial_eval(polynomial_mul(p, q), x) = F.0
    }
}

/// A point is a root of the product of `X - a` and `X - b` exactly when it
/// is `a` or `b`.
theorem polynomial_root_linear_product[F: Field](a: F, b: F, x: F) {
    (polynomial_eval(polynomial_mul(linear_polynomial(F.1, -a),
        linear_polynomial(F.1, -b)), x) = F.0) = (x = a or x = b)
} by {
    polynomial_root_product_iff(linear_polynomial(F.1, -a), linear_polynomial(F.1, -b), x)
    (polynomial_eval(polynomial_mul(linear_polynomial(F.1, -a),
        linear_polynomial(F.1, -b)), x) = F.0) =
        (polynomial_eval(linear_polynomial(F.1, -a), x) = F.0 or
            polynomial_eval(linear_polynomial(F.1, -b), x) = F.0)
    polynomial_root_linear_one(a, x)
    (polynomial_eval(linear_polynomial(F.1, -a), x) = F.0) = (x = a)
    polynomial_root_linear_one(b, x)
    (polynomial_eval(linear_polynomial(F.1, -b), x) = F.0) = (x = b)
    (polynomial_eval(polynomial_mul(linear_polynomial(F.1, -a),
        linear_polynomial(F.1, -b)), x) = F.0) = (x = a or x = b)
}

/// A root at `r` translates to a root of the shifted evaluation at `r - a`.
theorem polynomial_shift_root[R: CommRing](p: Polynomial[R], a: R, r: R) {
    (polynomial_eval(p, r) = R.0) = (polynomial_shift_eval(p, a, r - a) = R.0)
} by {
    polynomial_shift_eval(p, a, r - a) = polynomial_eval(p, (r - a) + a)
    roots_add_sub_cancel(a, r)
    (r - a) + a = r
    polynomial_eval(p, (r - a) + a) = polynomial_eval(p, r)
    polynomial_shift_eval(p, a, r - a) = polynomial_eval(p, r)
    (polynomial_eval(p, r) = R.0) = (polynomial_shift_eval(p, a, r - a) = R.0)
}

/// Scaling evaluation by `c` at `x` evaluates at `c * x`.
theorem polynomial_scale_eval_of_mul[R: CommRing](p: Polynomial[R], c: R, x: R) {
    polynomial_scale_eval(p, c, x) = polynomial_eval(p, c * x)
} by {
    polynomial_scale_eval(p, c, x) = polynomial_eval(p, c * x)
}

/// Over a field, a root `r` of `p` scales to a root of the scaled
/// evaluation: `p(c * (r / c))` vanishes.
theorem polynomial_root_scale_field[F: Field](p: Polynomial[F], c: F, r: F) {
    c != F.0 implies
    (polynomial_eval(p, r) = F.0) = (polynomial_scale_eval(p, c, r * c.inverse) = F.0)
} by {
    if c != F.0 {
        polynomial_scale_eval(p, c, r * c.inverse) = polynomial_eval(p, c * (r * c.inverse))
        Semigroup.mul_associative[F](c, r, c.inverse)
        c * (r * c.inverse) = (c * r) * c.inverse
        CommSemigroup.commutative[F](c, r)
        c * r = r * c
        (c * r) * c.inverse = (r * c) * c.inverse
        Semigroup.mul_associative[F](r, c, c.inverse)
        r * (c * c.inverse) = (r * c) * c.inverse
        mul_inverse_right(c)
        c * c.inverse = F.1
        r * (c * c.inverse) = r * F.1
        r * F.1 = r
        c * (r * c.inverse) = r
        polynomial_eval(p, c * (r * c.inverse)) = polynomial_eval(p, r)
        polynomial_scale_eval(p, c, r * c.inverse) = polynomial_eval(p, r)
        (polynomial_eval(p, r) = F.0) = (polynomial_scale_eval(p, c, r * c.inverse) = F.0)
    }
}

/// A point is a root of `p` exactly when zero is a root of the evaluation
/// shifted by that point.
theorem polynomial_root_shift_zero[R: CommRing](p: Polynomial[R], a: R) {
    (polynomial_eval(p, a) = R.0) = (polynomial_shift_eval(p, a, R.0) = R.0)
} by {
    polynomial_shift_eval(p, a, R.0) = polynomial_eval(p, R.0 + a)
    R.0 + a = a
    polynomial_eval(p, R.0 + a) = polynomial_eval(p, a)
    polynomial_shift_eval(p, a, R.0) = polynomial_eval(p, a)
    (polynomial_eval(p, a) = R.0) = (polynomial_shift_eval(p, a, R.0) = R.0)
}

/// Evaluating at zero gives a root exactly when the constant coefficient
/// vanishes.
theorem polynomial_root_zero_point[R: Semiring](p: Polynomial[R]) {
    (polynomial_eval(p, R.0) = R.0) = (p.coeff(Nat.0) = R.0)
} by {
    polynomial_eval_zero_point(p)
    polynomial_eval(p, R.0) = p.coeff(Nat.0)
    (polynomial_eval(p, R.0) = R.0) = (p.coeff(Nat.0) = R.0)
}

/// A monomial of positive degree vanishes at a point exactly when its
/// coefficient or the power of the point vanishes.
theorem polynomial_root_monomial_power[F: Field](n: Nat, r: F, x: F) {
    (polynomial_eval(polynomial_monomial(n.suc, r), x) = F.0) =
        (r = F.0 or x.pow(n.suc) = F.0)
} by {
    polynomial_eval_monomial(n.suc, r, x)
    polynomial_eval(polynomial_monomial(n.suc, r), x) = r * x.pow(n.suc)
    if polynomial_eval(polynomial_monomial(n.suc, r), x) = F.0 {
        r * x.pow(n.suc) = F.0
        field_mul_eq_zero(r, x.pow(n.suc))
        r = F.0 or x.pow(n.suc) = F.0
    }
    if r = F.0 or x.pow(n.suc) = F.0 {
        if r = F.0 {
            r * x.pow(n.suc) = F.0 * x.pow(n.suc)
            F.0 * x.pow(n.suc) = F.0
            r * x.pow(n.suc) = F.0
        } else {
            x.pow(n.suc) = F.0
            r * x.pow(n.suc) = r * F.0
            r * F.0 = F.0
            r * x.pow(n.suc) = F.0
        }
        polynomial_eval(polynomial_monomial(n.suc, r), x) = F.0
    }
}
