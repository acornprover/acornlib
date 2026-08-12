/// Divisibility of polynomials: the `p | q` predicate and its algebraic
/// properties (reflexivity, transitivity, and closure under sums, differences
/// and products).

from nat import Nat
from comm_ring import CommRing
from algebra.ring.ring import Ring
from polynomial import Polynomial, polynomial_zero_coeff, polynomial_ext_pointwise,
    polynomial_mul, polynomial_mul_coeff, polynomial_mul_coeff_apply,
    polynomial_mul_term_coeff, polynomial_eval, polynomial_eval_mul
from data.nat.nat_range_sum import range_sum, range_sum_congr
from data.nat.nat_residue_range_sum import range_sum_zero_of_vanishes
from polynomial_mul_comm import mul_coeff_eq_range_sum, polynomial_mul_comm,
    polynomial_mul_one_right, polynomial_mul_one_left
from polynomial_mul_assoc import polynomial_mul_assoc
from polynomial_mul_distrib import polynomial_mul_add_left
from polynomial.polynomial_ring_theory import polynomial_mul_add_right, polynomial_mul_neg_right

numerals Nat

/// A polynomial `p` divides `q` when some polynomial `r` witnesses
/// `q = p * r`.
define polynomial_divides[R: CommRing](p: Polynomial[R], q: Polynomial[R]) -> Bool {
    exists(r: Polynomial[R]) {
        polynomial_mul(p, r) = q
    }
}

/// One summand of a left zero product vanishes.
lemma mul_zero_left_term_vanishes[R: CommRing](p: Polynomial[R], k: Nat, i: Nat) {
    i < k.suc implies polynomial_mul_term_coeff(Polynomial[R].zero, p, k, i) = R.0
} by {
    if i < k.suc {
        polynomial_mul_term_coeff(Polynomial[R].zero, p, k, i) =
            Polynomial[R].zero.coeff(i) * p.coeff(k - i)
        polynomial_zero_coeff[R](i)
        Polynomial[R].zero.coeff(i) = R.0
        polynomial_mul_term_coeff(Polynomial[R].zero, p, k, i) = R.0 * p.coeff(k - i)
        R.0 * p.coeff(k - i) = R.0
    }
}

/// One summand of a right zero product vanishes.
lemma mul_zero_right_term_vanishes[R: CommRing](p: Polynomial[R], k: Nat, i: Nat) {
    i < k.suc implies polynomial_mul_term_coeff(p, Polynomial[R].zero, k, i) = R.0
} by {
    if i < k.suc {
        polynomial_mul_term_coeff(p, Polynomial[R].zero, k, i) =
            p.coeff(i) * Polynomial[R].zero.coeff(k - i)
        polynomial_zero_coeff[R](k - i)
        Polynomial[R].zero.coeff(k - i) = R.0
        polynomial_mul_term_coeff(p, Polynomial[R].zero, k, i) = p.coeff(i) * R.0
        p.coeff(i) * R.0 = R.0
    }
}

/// The zero polynomial absorbs products on the left.
theorem polynomial_mul_zero_left[R: CommRing](p: Polynomial[R]) {
    polynomial_mul(Polynomial[R].zero, p) = Polynomial[R].zero
} by {
    forall(k: Nat) {
        polynomial_mul_coeff_apply(Polynomial[R].zero, p, k)
        polynomial_mul(Polynomial[R].zero, p).coeff(k) = polynomial_mul_coeff(Polynomial[R].zero, p, k)
        mul_coeff_eq_range_sum(Polynomial[R].zero, p, k)
        polynomial_mul_coeff(Polynomial[R].zero, p, k) =
            range_sum(polynomial_mul_term_coeff(Polynomial[R].zero, p, k), k.suc)
        forall(i: Nat) {
            if i < k.suc {
                mul_zero_left_term_vanishes(p, k, i)
                polynomial_mul_term_coeff(Polynomial[R].zero, p, k, i) = R.0
            }
        }
        range_sum_zero_of_vanishes(polynomial_mul_term_coeff(Polynomial[R].zero, p, k), k.suc)
        range_sum(polynomial_mul_term_coeff(Polynomial[R].zero, p, k), k.suc) = R.0
        polynomial_mul_coeff(Polynomial[R].zero, p, k) = R.0
        polynomial_mul(Polynomial[R].zero, p).coeff(k) = R.0
        polynomial_zero_coeff[R](k)
        polynomial_mul(Polynomial[R].zero, p).coeff(k) = Polynomial[R].zero.coeff(k)
    }
    polynomial_ext_pointwise(polynomial_mul(Polynomial[R].zero, p), Polynomial[R].zero)
}

/// The zero polynomial absorbs products on the right.
theorem polynomial_mul_zero_right[R: CommRing](p: Polynomial[R]) {
    polynomial_mul(p, Polynomial[R].zero) = Polynomial[R].zero
} by {
    forall(k: Nat) {
        polynomial_mul_coeff_apply(p, Polynomial[R].zero, k)
        polynomial_mul(p, Polynomial[R].zero).coeff(k) = polynomial_mul_coeff(p, Polynomial[R].zero, k)
        mul_coeff_eq_range_sum(p, Polynomial[R].zero, k)
        polynomial_mul_coeff(p, Polynomial[R].zero, k) =
            range_sum(polynomial_mul_term_coeff(p, Polynomial[R].zero, k), k.suc)
        forall(i: Nat) {
            if i < k.suc {
                mul_zero_right_term_vanishes(p, k, i)
                polynomial_mul_term_coeff(p, Polynomial[R].zero, k, i) = R.0
            }
        }
        range_sum_zero_of_vanishes(polynomial_mul_term_coeff(p, Polynomial[R].zero, k), k.suc)
        range_sum(polynomial_mul_term_coeff(p, Polynomial[R].zero, k), k.suc) = R.0
        polynomial_mul_coeff(p, Polynomial[R].zero, k) = R.0
        polynomial_mul(p, Polynomial[R].zero).coeff(k) = R.0
        polynomial_zero_coeff[R](k)
        polynomial_mul(p, Polynomial[R].zero).coeff(k) = Polynomial[R].zero.coeff(k)
    }
    polynomial_ext_pointwise(polynomial_mul(p, Polynomial[R].zero), Polynomial[R].zero)
}

/// Divisibility is reflexive: `p | p`.
theorem polynomial_divides_refl[R: CommRing](p: Polynomial[R]) {
    polynomial_divides(p, p)
} by {
    polynomial_mul_one_right(p)
    polynomial_mul(p, Polynomial[R].one) = p
    exists(r: Polynomial[R]) {
        r = Polynomial[R].one and polynomial_mul(p, r) = p
    }
    polynomial_divides(p, p)
}

/// Every polynomial divides the zero polynomial: `p | 0`.
theorem polynomial_divides_zero_right[R: CommRing](p: Polynomial[R]) {
    polynomial_divides(p, Polynomial[R].zero)
} by {
    polynomial_mul_zero_right(p)
    polynomial_mul(p, Polynomial[R].zero) = Polynomial[R].zero
    exists(r: Polynomial[R]) {
        r = Polynomial[R].zero and polynomial_mul(p, r) = Polynomial[R].zero
    }
    polynomial_divides(p, Polynomial[R].zero)
}

/// If the zero polynomial divides `p`, then `p` is zero.
theorem polynomial_divides_zero_left_imp[R: CommRing](p: Polynomial[R]) {
    polynomial_divides(Polynomial[R].zero, p) implies p = Polynomial[R].zero
} by {
    if polynomial_divides(Polynomial[R].zero, p) {
        let r: Polynomial[R] satisfy {
            polynomial_mul(Polynomial[R].zero, r) = p
        }
        polynomial_mul_zero_left(r)
        polynomial_mul(Polynomial[R].zero, r) = Polynomial[R].zero
        p = Polynomial[R].zero
    }
}

/// The zero polynomial divides exactly the zero polynomial.
theorem polynomial_divides_zero_iff[R: CommRing](p: Polynomial[R]) {
    polynomial_divides(Polynomial[R].zero, p) iff p = Polynomial[R].zero
} by {
    if polynomial_divides(Polynomial[R].zero, p) {
        polynomial_divides_zero_left_imp(p)
        p = Polynomial[R].zero
    }
    if p = Polynomial[R].zero {
        polynomial_divides_zero_right(Polynomial[R].zero)
        polynomial_divides(Polynomial[R].zero, Polynomial[R].zero)
        p = Polynomial[R].zero
        polynomial_divides(Polynomial[R].zero, p)
    }
}

/// Divisibility is preserved by adding a common multiple: if `d | p` and
/// `d | q` then `d | p + q`.
theorem polynomial_divides_add[R: CommRing](
    d: Polynomial[R], p: Polynomial[R], q: Polynomial[R]
) {
    polynomial_divides(d, p) and polynomial_divides(d, q) implies polynomial_divides(d, p + q)
} by {
    if polynomial_divides(d, p) and polynomial_divides(d, q) {
        let a: Polynomial[R] satisfy {
            polynomial_mul(d, a) = p
        }
        let b: Polynomial[R] satisfy {
            polynomial_mul(d, b) = q
        }
        polynomial_mul_add_left(d, a, b)
        polynomial_mul(d, a + b) = polynomial_mul(d, a) + polynomial_mul(d, b)
        polynomial_mul(d, a) = p
        polynomial_mul(d, b) = q
        polynomial_mul(d, a + b) = p + q
        exists(r: Polynomial[R]) {
            r = a + b and polynomial_mul(d, r) = p + q
        }
        polynomial_divides(d, p + q)
    }
}

/// Divisibility is preserved by multiplying on the right: if `d | p` then
/// `d | p * q`.
theorem polynomial_divides_mul_right[R: CommRing](
    d: Polynomial[R], p: Polynomial[R], q: Polynomial[R]
) {
    polynomial_divides(d, p) implies polynomial_divides(d, polynomial_mul(p, q))
} by {
    if polynomial_divides(d, p) {
        let a: Polynomial[R] satisfy {
            polynomial_mul(d, a) = p
        }
        polynomial_mul_assoc(d, a, q)
        polynomial_mul(polynomial_mul(d, a), q) = polynomial_mul(d, polynomial_mul(a, q))
        polynomial_mul(d, a) = p
        polynomial_mul(p, q) = polynomial_mul(d, polynomial_mul(a, q))
        exists(r: Polynomial[R]) {
            r = polynomial_mul(a, q) and polynomial_mul(d, r) = polynomial_mul(p, q)
        }
        polynomial_divides(d, polynomial_mul(p, q))
    }
}

/// Divisibility is preserved by multiplying on the left: if `d | p` then
/// `d | q * p`.
theorem polynomial_divides_mul_left[R: CommRing](
    d: Polynomial[R], p: Polynomial[R], q: Polynomial[R]
) {
    polynomial_divides(d, p) implies polynomial_divides(d, polynomial_mul(q, p))
} by {
    if polynomial_divides(d, p) {
        polynomial_divides_mul_right(d, p, q)
        polynomial_divides(d, polynomial_mul(p, q))
        polynomial_mul_comm(p, q)
        polynomial_mul(p, q) = polynomial_mul(q, p)
        polynomial_divides(d, polynomial_mul(q, p))
    }
}

/// Divisibility is transitive: if `d | e` and `e | f` then `d | f`.
theorem polynomial_divides_trans[R: CommRing](
    d: Polynomial[R], e: Polynomial[R], f: Polynomial[R]
) {
    polynomial_divides(d, e) and polynomial_divides(e, f) implies polynomial_divides(d, f)
} by {
    if polynomial_divides(d, e) and polynomial_divides(e, f) {
        let a: Polynomial[R] satisfy {
            polynomial_mul(d, a) = e
        }
        let b: Polynomial[R] satisfy {
            polynomial_mul(e, b) = f
        }
        polynomial_mul_assoc(d, a, b)
        polynomial_mul(polynomial_mul(d, a), b) = polynomial_mul(d, polynomial_mul(a, b))
        polynomial_mul(d, a) = e
        polynomial_mul(e, b) = polynomial_mul(polynomial_mul(d, a), b)
        polynomial_mul(e, b) = f
        polynomial_mul(d, polynomial_mul(a, b)) = f
        exists(r: Polynomial[R]) {
            r = polynomial_mul(a, b) and polynomial_mul(d, r) = f
        }
        polynomial_divides(d, f)
    }
}

/// Divisibility is preserved by taking differences: if `d | p` and `d | q`
/// then `d | p - q`.
theorem polynomial_divides_sub[R: CommRing](
    d: Polynomial[R], p: Polynomial[R], q: Polynomial[R]
) {
    polynomial_divides(d, p) and polynomial_divides(d, q) implies polynomial_divides(d, p.sub(q))
} by {
    if polynomial_divides(d, p) and polynomial_divides(d, q) {
        let a: Polynomial[R] satisfy {
            polynomial_mul(d, a) = p
        }
        let b: Polynomial[R] satisfy {
            polynomial_mul(d, b) = q
        }
        polynomial_mul_neg_right(d, b)
        polynomial_mul(d, b.neg) = polynomial_mul(d, b).neg
        polynomial_mul(d, b) = q
        polynomial_mul(d, b.neg) = q.neg
        polynomial_divides_add(d, p, q.neg)
        polynomial_divides(d, p + q.neg)
        p.sub(q) = p + q.neg
        polynomial_divides(d, p.sub(q))
    }
}

/// A divisibility witness forces divisibility of values at one point.
theorem polynomial_divides_eval_witness[R: CommRing](
    d: Polynomial[R], p: Polynomial[R], r: Polynomial[R], x: R
) {
    polynomial_mul(d, r) = p implies
    polynomial_eval(p, x) = polynomial_eval(d, x) * polynomial_eval(r, x)
} by {
    if polynomial_mul(d, r) = p {
        polynomial_eval_mul(d, r, x)
        polynomial_eval(polynomial_mul(d, r), x) = polynomial_eval(d, x) * polynomial_eval(r, x)
        polynomial_eval(p, x) = polynomial_eval(polynomial_mul(d, r), x)
        polynomial_eval(p, x) = polynomial_eval(d, x) * polynomial_eval(r, x)
    }
}

/// Divisibility forces divisibility of values: if `d | p` then at every point
/// the value of `p` is the value of `d` times the value of the quotient.
theorem polynomial_divides_eval[R: CommRing](
    d: Polynomial[R], p: Polynomial[R], x: R
) {
    polynomial_divides(d, p) implies
    exists(r: Polynomial[R]) {
        polynomial_eval(p, x) = polynomial_eval(d, x) * polynomial_eval(r, x)
    }
} by {
    if polynomial_divides(d, p) {
        let r: Polynomial[R] satisfy {
            polynomial_mul(d, r) = p
        }
        polynomial_divides_eval_witness(d, p, r, x)
        polynomial_eval(p, x) = polynomial_eval(d, x) * polynomial_eval(r, x)
        exists(s: Polynomial[R]) {
            s = r and polynomial_eval(p, x) = polynomial_eval(d, x) * polynomial_eval(s, x)
        }
        exists(s: Polynomial[R]) {
            polynomial_eval(p, x) = polynomial_eval(d, x) * polynomial_eval(s, x)
        }
    }
}
