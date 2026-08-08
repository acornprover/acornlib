/// Bridge from finitely supported coefficient functions to bounded coefficient evaluation.

from algebra.add_monoid_algebra import AddMonoidAlgebra
from finite_set import FiniteSet
from algebra.module.finite_support import has_support_in, has_support_in_apply_zero
from nat import Nat
from nat import alt_induction, lt_and_lte, lt_cancel_suc, lt_or_lte, not_lt_zero
from polynomial.coeff_eval import coeff_eval, coeff_tail
from semiring import Semiring

numerals Nat

/// The assertion that a coefficient function vanishes at one index.
define coeff_zero_at[R: Semiring](c: Nat -> R, k: Nat) -> Bool {
    c(k) = R.0
}

/// The implication that one coefficient outside a bound is zero.
define coeff_zero_from_at[R: Semiring](c: Nat -> R, n: Nat, k: Nat) -> Bool {
    not k < n implies coeff_zero_at(c, k)
}

/// A coefficient function is zero from the bound onward.
define coeff_zero_from[R: Semiring](c: Nat -> R, n: Nat) -> Bool {
    forall(k: Nat) {
        coeff_zero_from_at(c, n, k)
    }
}

/// One more zero coefficient past a bound does not change bounded Horner evaluation.
define coeff_eval_stable_suc_at[R: Semiring](c: Nat -> R, x: R, n: Nat) -> Bool {
    coeff_zero_from(c, n) implies coeff_eval(c, x, n.suc) = coeff_eval(c, x, n)
}

/// The finite-support polynomial represented by an additive monoid algebra has no terms at or above `n`.
define add_monoid_algebra_support_bounded_by[R: Semiring](p: AddMonoidAlgebra[R, Nat], n: Nat) -> Bool {
    coeff_zero_from(p.coeff, n)
}

/// Bounded evaluation of a finite-support polynomial representative.
define add_monoid_algebra_eval_bound[R: Semiring](p: AddMonoidAlgebra[R, Nat], x: R, n: Nat) -> R {
    coeff_eval(p.coeff, x, n)
}

/// The assertion that a finite set has no exponent outside a strict bound at one index.
define finite_set_nat_outside_bound(s: FiniteSet[Nat], n: Nat, k: Nat) -> Bool {
    not k < n implies not s.contains(k)
}

/// A finite set of natural-number exponents is strictly bounded by `n`.
define finite_set_nat_bounded_by(s: FiniteSet[Nat], n: Nat) -> Bool {
    forall(k: Nat) {
        finite_set_nat_outside_bound(s, n, k)
    }
}

lemma nat_lte_add_right(a: Nat, b: Nat) {
    a <= a + b
} by {
    a + b = a + b
}

lemma coeff_zero_from_at_intro[R: Semiring](c: Nat -> R, n: Nat, k: Nat) {
    not k < n and c(k) = R.0 implies coeff_zero_from_at(c, n, k)
} by {
    if not k < n and c(k) = R.0 {
        coeff_zero_at(c, k)
        coeff_zero_from_at(c, n, k) = (not k < n implies coeff_zero_at(c, k))
        coeff_zero_from_at(c, n, k)
    }
}

lemma finite_set_nat_outside_bound_intro(s: FiniteSet[Nat], n: Nat, k: Nat) {
    not k < n and not s.contains(k) implies finite_set_nat_outside_bound(s, n, k)
} by {
    if not k < n and not s.contains(k) {
        finite_set_nat_outside_bound(s, n, k) = (not k < n implies not s.contains(k))
        finite_set_nat_outside_bound(s, n, k)
    }
}

lemma coeff_zero_from_intro[R: Semiring](c: Nat -> R, n: Nat) {
    (forall(k: Nat) { coeff_zero_from_at(c, n, k) }) implies coeff_zero_from(c, n)
} by {
    forall(k: Nat) {
        coeff_zero_from_at(c, n, k)
    }
    coeff_zero_from(c, n) = forall(k: Nat) {
        coeff_zero_from_at(c, n, k)
    }
}

/// Applying a zero-from-bound witness at an index outside the bound gives a zero coefficient.
theorem coeff_zero_from_apply[R: Semiring](c: Nat -> R, n: Nat, k: Nat) {
    coeff_zero_from(c, n) and not k < n implies c(k) = R.0
} by {
    if coeff_zero_from(c, n) and not k < n {
        coeff_zero_from(c, n) = forall(i: Nat) {
            coeff_zero_from_at(c, n, i)
        }
        coeff_zero_from_at(c, n, k)
        coeff_zero_from_at(c, n, k) = (not k < n implies coeff_zero_at(c, k))
        coeff_zero_at(c, k)
        coeff_zero_at(c, k) = (c(k) = R.0)
        c(k) = R.0
    }
}

/// A finite-set bound excludes any exponent outside the bound.
theorem finite_set_nat_bounded_by_not_contains(s: FiniteSet[Nat], n: Nat, k: Nat) {
    finite_set_nat_bounded_by(s, n) and not k < n implies not s.contains(k)
} by {
    if finite_set_nat_bounded_by(s, n) and not k < n {
        finite_set_nat_bounded_by(s, n) = forall(i: Nat) {
            finite_set_nat_outside_bound(s, n, i)
        }
        finite_set_nat_outside_bound(s, n, k)
        finite_set_nat_outside_bound(s, n, k) = (not k < n implies not s.contains(k))
        not s.contains(k)
    }
}

/// Applying a finite-set exponent bound at a member gives the strict bound.
theorem finite_set_nat_bounded_by_apply(s: FiniteSet[Nat], n: Nat, k: Nat) {
    finite_set_nat_bounded_by(s, n) and s.contains(k) implies k < n
} by {
    if finite_set_nat_bounded_by(s, n) and s.contains(k) {
        if not k < n {
            finite_set_nat_bounded_by_not_contains(s, n, k)
            not s.contains(k)
            false
        }
        k < n
    }
}

lemma not_suc_lt_suc_of_not_lt(a: Nat, b: Nat) {
    not a < b implies not a.suc < b.suc
} by {
    if not a < b {
        if a.suc < b.suc {
            lt_cancel_suc(a, b)
            a < b
            false
        }
    }
}

lemma coeff_zero_from_tail[R: Semiring](c: Nat -> R, n: Nat) {
    coeff_zero_from(c, n.suc) implies coeff_zero_from(coeff_tail(c), n)
} by {
    if coeff_zero_from(c, n.suc) {
        forall(k: Nat) {
            if not k < n {
                not_suc_lt_suc_of_not_lt(k, n)
                not k.suc < n.suc
                coeff_zero_from_apply(c, n.suc, k.suc)
                c(k.suc) = R.0
                coeff_tail(c, k) = c(k.suc)
                coeff_tail(c, k) = R.0
                coeff_zero_at(coeff_tail(c), k)
                coeff_zero_from_at_intro(coeff_tail(c), n, k)
                coeff_zero_from_at(coeff_tail(c), n, k)
            }
        }
    }
}

lemma coeff_zero_from_monotone[R: Semiring](c: Nat -> R, n: Nat, m: Nat) {
    coeff_zero_from(c, n) and n <= m implies coeff_zero_from(c, m)
} by {
    if coeff_zero_from(c, n) and n <= m {
        forall(k: Nat) {
            if not k < m {
                if k < n {
                    lt_and_lte(k, n, m)
                    k < m
                    false
                }
                not k < n
                coeff_zero_from_apply(c, n, k)
                c(k) = R.0
                coeff_zero_at(c, k)
                coeff_zero_from_at_intro(c, m, k)
                coeff_zero_from_at(c, m, k)
            }
        }
    }
}

/// The statement that one more zero coefficient past a bound does not change evaluation.
define coeff_eval_stable_suc_property[R: Semiring](x: R, n: Nat) -> Bool {
    forall(c: Nat -> R) {
        coeff_eval_stable_suc_at(c, x, n)
    }
}

lemma coeff_eval_stable_suc_property_apply[R: Semiring](x: R, n: Nat, c: Nat -> R) {
    coeff_eval_stable_suc_property(x, n) and coeff_zero_from(c, n) implies
    coeff_eval(c, x, n.suc) = coeff_eval(c, x, n)
} by {
    if coeff_eval_stable_suc_property(x, n) and coeff_zero_from(c, n) {
        coeff_eval_stable_suc_property(x, n) = forall(d: Nat -> R) {
            coeff_eval_stable_suc_at(d, x, n)
        }
        coeff_eval_stable_suc_at(c, x, n)
        coeff_eval_stable_suc_at(c, x, n) =
            (coeff_zero_from(c, n) implies coeff_eval(c, x, n.suc) = coeff_eval(c, x, n))
        coeff_eval(c, x, n.suc) = coeff_eval(c, x, n)
    }
}

lemma coeff_eval_stable_suc_property_intro[R: Semiring](x: R, n: Nat) {
    (forall(c: Nat -> R) {
        coeff_eval_stable_suc_at(c, x, n)
    }) implies coeff_eval_stable_suc_property(x, n)
} by {
    forall(c: Nat -> R) {
        coeff_eval_stable_suc_at(c, x, n)
    }
    coeff_eval_stable_suc_property(x, n) = forall(c: Nat -> R) {
        coeff_eval_stable_suc_at(c, x, n)
    }
}

lemma coeff_eval_stable_suc_property_all[R: Semiring](x: R, n: Nat) {
    coeff_eval_stable_suc_property(x, n)
} by {
    define statement(bound: Nat) -> Bool {
        coeff_eval_stable_suc_property(x, bound)
    }

    forall(c: Nat -> R) {
        if coeff_zero_from(c, Nat.0) {
            not_lt_zero(Nat.0)
            coeff_zero_from_apply(c, Nat.0, Nat.0)
            c(Nat.0) = R.0
            coeff_eval(c, x, Nat.0) = R.0
            coeff_eval(coeff_tail(c), x, Nat.0) = R.0
            x * R.0 = R.0
            c(Nat.0) + x * coeff_eval(coeff_tail(c), x, Nat.0) = R.0
            coeff_eval(c, x, Nat.0.suc) = c(Nat.0) + x * coeff_eval(coeff_tail(c), x, Nat.0)
            coeff_eval(c, x, Nat.0.suc) = R.0
            coeff_eval(c, x, Nat.0.suc) = coeff_eval(c, x, Nat.0)
        }
        coeff_eval_stable_suc_at(c, x, Nat.0) =
            (coeff_zero_from(c, Nat.0) implies coeff_eval(c, x, Nat.0.suc) = coeff_eval(c, x, Nat.0))
        coeff_eval_stable_suc_at(c, x, Nat.0)
    }
    coeff_eval_stable_suc_property_intro(x, Nat.0)
    coeff_eval_stable_suc_property(x, Nat.0)
    statement(Nat.0)

    forall(bound: Nat) {
        if statement(bound) {
            forall(c: Nat -> R) {
                if coeff_zero_from(c, bound.suc) {
                    coeff_zero_from_tail(c, bound)
                    coeff_zero_from(coeff_tail(c), bound)
                    statement(bound) = coeff_eval_stable_suc_property(x, bound)
                    coeff_eval_stable_suc_property_apply(x, bound, coeff_tail(c))
                    coeff_eval(coeff_tail(c), x, bound.suc) = coeff_eval(coeff_tail(c), x, bound)
                    coeff_eval(c, x, bound.suc.suc) = c(Nat.0) + x * coeff_eval(coeff_tail(c), x, bound.suc)
                    coeff_eval(c, x, bound.suc) = c(Nat.0) + x * coeff_eval(coeff_tail(c), x, bound)
                    coeff_eval(c, x, bound.suc.suc) = coeff_eval(c, x, bound.suc)
                }
                coeff_eval_stable_suc_at(c, x, bound.suc) =
                    (coeff_zero_from(c, bound.suc) implies
                    coeff_eval(c, x, bound.suc.suc) = coeff_eval(c, x, bound.suc))
                coeff_eval_stable_suc_at(c, x, bound.suc)
            }
            coeff_eval_stable_suc_property_intro(x, bound.suc)
            coeff_eval_stable_suc_property(x, bound.suc)
            statement(bound.suc)
        }
    }

    alt_induction(statement)
    statement(n)
}

/// Adding one more zero coefficient past a bound does not change bounded Horner evaluation.
theorem coeff_eval_stable_suc_of_zero_from[R: Semiring](c: Nat -> R, x: R, n: Nat) {
    coeff_zero_from(c, n) implies coeff_eval(c, x, n.suc) = coeff_eval(c, x, n)
} by {
    if coeff_zero_from(c, n) {
        coeff_eval_stable_suc_property_all(x, n)
        coeff_eval_stable_suc_property_apply(x, n, c)
    }
}

/// Adding only zero coefficients past a bound does not change bounded Horner evaluation.
theorem coeff_eval_stable_add_of_zero_from[R: Semiring](c: Nat -> R, x: R, n: Nat, d: Nat) {
    coeff_zero_from(c, n) implies coeff_eval(c, x, n + d) = coeff_eval(c, x, n)
} by {
    if coeff_zero_from(c, n) {
        define statement(offset: Nat) -> Bool {
            coeff_eval(c, x, n + offset) = coeff_eval(c, x, n)
        }

        n + Nat.0 = n
        statement(Nat.0)

        forall(offset: Nat) {
            if statement(offset) {
                nat_lte_add_right(n, offset)
                n <= n + offset
                coeff_zero_from_monotone(c, n, n + offset)
                coeff_zero_from(c, n + offset)
                coeff_eval_stable_suc_of_zero_from(c, x, n + offset)
                coeff_eval(c, x, (n + offset).suc) = coeff_eval(c, x, n + offset)
                n + offset.suc = (n + offset).suc
                coeff_eval(c, x, n + offset.suc) = coeff_eval(c, x, n + offset)
                coeff_eval(c, x, n + offset) = coeff_eval(c, x, n)
                coeff_eval(c, x, n + offset.suc) = coeff_eval(c, x, n)
                statement(offset.suc)
            }
        }

        alt_induction(statement)
        statement(d)
        coeff_eval(c, x, n + d) = coeff_eval(c, x, n)
    }
}

/// Bounded Horner evaluation is independent of any larger bound once the coefficients vanish.
theorem coeff_eval_eq_of_zero_from_le[R: Semiring](c: Nat -> R, x: R, n: Nat, m: Nat) {
    coeff_zero_from(c, n) and n <= m implies coeff_eval(c, x, m) = coeff_eval(c, x, n)
} by {
    if coeff_zero_from(c, n) and n <= m {
        let d: Nat satisfy {
            n + d = m
        }
        coeff_eval_stable_add_of_zero_from(c, x, n, d)
        coeff_eval(c, x, n + d) = coeff_eval(c, x, n)
        coeff_eval(c, x, m) = coeff_eval(c, x, n)
    }
}

/// A finite support contained below `n` gives a polynomial support bound at `n`.
theorem add_monoid_algebra_support_bounded_by_of_has_support_in[R: Semiring](
    p: AddMonoidAlgebra[R, Nat],
    s: FiniteSet[Nat],
    n: Nat
) {
    has_support_in(p.coeff, s) and finite_set_nat_bounded_by(s, n) implies
    add_monoid_algebra_support_bounded_by(p, n)
} by {
    if has_support_in(p.coeff, s) and finite_set_nat_bounded_by(s, n) {
        forall(k: Nat) {
            if k < n {
                coeff_zero_from_at(p.coeff, n, k) = (not k < n implies coeff_zero_at(p.coeff, k))
                coeff_zero_from_at(p.coeff, n, k)
            } else {
                not k < n
                finite_set_nat_bounded_by_not_contains(s, n, k)
                not s.contains(k)
                has_support_in_apply_zero[Nat, R](p.coeff, s, k)
                p.coeff(k) = R.0
                coeff_zero_at(p.coeff, k)
                coeff_zero_from_at_intro(p.coeff, n, k)
                coeff_zero_from_at(p.coeff, n, k)
            }
        }
        coeff_zero_from_intro(p.coeff, n)
        coeff_zero_from(p.coeff, n)
        add_monoid_algebra_support_bounded_by(p, n)
    }
}

/// Bounded polynomial evaluation is independent of increasing a valid support bound.
theorem add_monoid_algebra_eval_bound_eq_of_support_bounded_le[R: Semiring](
    p: AddMonoidAlgebra[R, Nat],
    x: R,
    n: Nat,
    m: Nat
) {
    add_monoid_algebra_support_bounded_by(p, n) and n <= m implies
    add_monoid_algebra_eval_bound(p, x, m) = add_monoid_algebra_eval_bound(p, x, n)
} by {
    if add_monoid_algebra_support_bounded_by(p, n) and n <= m {
        coeff_eval_eq_of_zero_from_le(p.coeff, x, n, m)
        coeff_eval(p.coeff, x, m) = coeff_eval(p.coeff, x, n)
        add_monoid_algebra_eval_bound(p, x, m) = coeff_eval(p.coeff, x, m)
        add_monoid_algebra_eval_bound(p, x, n) = coeff_eval(p.coeff, x, n)
        add_monoid_algebra_eval_bound(p, x, m) = add_monoid_algebra_eval_bound(p, x, n)
    }
}

/// A concrete finite support bound gives independence after enlarging the evaluation bound.
theorem add_monoid_algebra_eval_bound_eq_of_has_support_in_le[R: Semiring](
    p: AddMonoidAlgebra[R, Nat],
    s: FiniteSet[Nat],
    x: R,
    n: Nat,
    m: Nat
) {
    has_support_in(p.coeff, s) and finite_set_nat_bounded_by(s, n) and n <= m implies
    add_monoid_algebra_eval_bound(p, x, m) = add_monoid_algebra_eval_bound(p, x, n)
} by {
    if has_support_in(p.coeff, s) and finite_set_nat_bounded_by(s, n) and n <= m {
        add_monoid_algebra_support_bounded_by_of_has_support_in(p, s, n)
        add_monoid_algebra_support_bounded_by(p, n)
        add_monoid_algebra_eval_bound_eq_of_support_bounded_le(p, x, n, m)
        add_monoid_algebra_eval_bound(p, x, m) = add_monoid_algebra_eval_bound(p, x, n)
    }
}

/// Bounded polynomial evaluation is independent of the chosen valid support bound.
theorem add_monoid_algebra_eval_bound_eq_of_support_bounded[R: Semiring](
    p: AddMonoidAlgebra[R, Nat],
    x: R,
    n: Nat,
    m: Nat
) {
    add_monoid_algebra_support_bounded_by(p, n) and add_monoid_algebra_support_bounded_by(p, m) implies
    add_monoid_algebra_eval_bound(p, x, n) = add_monoid_algebra_eval_bound(p, x, m)
} by {
    if add_monoid_algebra_support_bounded_by(p, n) and add_monoid_algebra_support_bounded_by(p, m) {
        if n <= m {
            add_monoid_algebra_eval_bound_eq_of_support_bounded_le(p, x, n, m)
            add_monoid_algebra_eval_bound(p, x, m) = add_monoid_algebra_eval_bound(p, x, n)
        } else {
            lt_or_lte(n, m)
            m <= n
            add_monoid_algebra_eval_bound_eq_of_support_bounded_le(p, x, m, n)
            add_monoid_algebra_eval_bound(p, x, n) = add_monoid_algebra_eval_bound(p, x, m)
        }
    }
}
