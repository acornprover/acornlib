/// Ring-theoretic structure of polynomials: the embedding of the coefficient
/// ring by constant polynomials, scalar multiplication, and the algebraic
/// laws (associativity, commutativity, distributivity) of polynomial
/// multiplication stated in coefficient form.

from nat import Nat, alt_induction, alt_suc_ne_zero, lt_suc, lt_imp_lt_suc
from semiring import Semiring
from comm_ring import CommRing
from algebra.ring.ring import Ring, mul_neg_right
from algebra.add_group import inverse_add
from list import partial, partial_one, partial_split_last, partial_zero, partial_pointwise_eq
from polynomial import Polynomial, polynomial_constant, polynomial_zero_coeff,
    polynomial_ext_pointwise, polynomial_add_coeff, polynomial_eq_coeff_at,
    polynomial_constant_coeff_zero, polynomial_constant_coeff_of_ne_zero,
    polynomial_neg_coeff, polynomial_one_coeff_zero, polynomial_one_coeff_of_ne_zero,
    polynomial_mul, polynomial_mul_coeff, polynomial_mul_coeff_apply, polynomial_mul_term_coeff
from polynomial_mul_comm import polynomial_mul_comm, mul_coeff_eq_range_sum
from polynomial_mul_distrib import polynomial_mul_add_left
from polynomial_mul_assoc import polynomial_mul_assoc

numerals Nat

/// A coefficient function that negates another pointwise.
define polynomial_neg_fn[R: Ring](c: Nat -> R, i: Nat) -> R {
    -c(i)
}

/// A partial sum whose summands all vanish within the bound is zero.
lemma polynomial_partial_zero_of_pointwise_zero[R: Semiring](f: Nat -> R, n: Nat) {
    (forall(i: Nat) { i < n implies f(i) = R.0 }) implies partial(f, n) = R.0
} by {
    define statement(k: Nat) -> Bool {
        forall(g: Nat -> R) {
            (forall(i: Nat) { i < k implies g(i) = R.0 }) implies partial(g, k) = R.0
        }
    }

    forall(g: Nat -> R) {
        partial_zero[R](g)
        partial(g, Nat.0) = R.0
    }
    statement(Nat.0)

    forall(k: Nat) {
        if statement(k) {
            forall(g: Nat -> R) {
                if forall(i: Nat) { i < k.suc implies g(i) = R.0 } {
                    partial_split_last[R](g, k)
                    partial(g, k.suc) = partial(g, k) + g(k)
                    forall(i: Nat) {
                        if i < k {
                            lt_imp_lt_suc(i, k)
                            i < k.suc
                            g(i) = R.0
                        }
                    }
                    statement(k) = forall(h: Nat -> R) {
                        (forall(j: Nat) { j < k implies h(j) = R.0 }) implies partial(h, k) = R.0
                    }
                    partial(g, k) = R.0
                    lt_suc(k)
                    k < k.suc
                    g(k) = R.0
                    partial(g, k.suc) = R.0 + R.0
                    R.0 + R.0 = R.0
                    partial(g, k.suc) = R.0
                }
            }
            statement(k.suc)
        }
    }

    statement(Nat.0) and forall(k: Nat) { statement(k) implies statement(k.suc) }
    alt_induction(statement)
    forall(k: Nat) { statement(k) }
    statement(n)
    statement(n) = forall(g: Nat -> R) {
        (forall(i: Nat) { i < n implies g(i) = R.0 }) implies partial(g, n) = R.0
    }
    partial(f, n) = R.0
}

/// A partial sum in which every summand after the first vanishes equals the
/// first summand.
lemma polynomial_partial_first_only[R: Semiring](f: Nat -> R, n: Nat) {
    (forall(i: Nat) { i < n.suc implies (i != Nat.0 implies f(i) = R.0) }) implies
    partial(f, n.suc) = f(Nat.0)
} by {
    define statement(m: Nat) -> Bool {
        (forall(i: Nat) { i < m.suc implies (i != Nat.0 implies f(i) = R.0) }) implies
        partial(f, m.suc) = f(Nat.0)
    }

    if forall(i: Nat) { i < Nat.0.suc implies (i != Nat.0 implies f(i) = R.0) } {
        partial_one[R](f)
        partial(f, Nat.0.suc) = f(Nat.0)
    }
    statement(Nat.0)

    forall(m: Nat) {
        if statement(m) {
            if forall(i: Nat) { i < m.suc.suc implies (i != Nat.0 implies f(i) = R.0) } {
                partial_split_last[R](f, m.suc)
                partial(f, m.suc.suc) = partial(f, m.suc) + f(m.suc)
                forall(i: Nat) {
                    if i < m.suc {
                        lt_imp_lt_suc(i, m.suc)
                        i < m.suc.suc
                        if i != Nat.0 {
                            f(i) = R.0
                        }
                    }
                }
                statement(m) =
                    ((forall(i: Nat) { i < m.suc implies (i != Nat.0 implies f(i) = R.0) }) implies
                    partial(f, m.suc) = f(Nat.0))
                partial(f, m.suc) = f(Nat.0)
                lt_suc(m.suc)
                m.suc < m.suc.suc
                alt_suc_ne_zero(m)
                m.suc != Nat.0
                f(m.suc) = R.0
                partial(f, m.suc.suc) = f(Nat.0) + R.0
                f(Nat.0) + R.0 = f(Nat.0)
                partial(f, m.suc.suc) = f(Nat.0)
            }
            statement(m.suc)
        }
    }

    statement(Nat.0) and forall(m: Nat) { statement(m) implies statement(m.suc) }
    alt_induction(statement)
    forall(m: Nat) { statement(m) }
    if forall(i: Nat) { i < n.suc implies (i != Nat.0 implies f(i) = R.0) } {
        statement(n)
        statement(n) = ((forall(i: Nat) { i < n.suc implies (i != Nat.0 implies f(i) = R.0) }) implies
            partial(f, n.suc) = f(Nat.0))
        partial(f, n.suc) = f(Nat.0)
    }
}

/// A helper: two functions agreeing on the first `n` indices have equal
/// partial sums over `n` summands.
lemma polynomial_function_extensionality_nat[R: Semiring](f: Nat -> R, g: Nat -> R, n: Nat) {
    (forall(i: Nat) { i < n implies f(i) = g(i) }) implies partial(f, n) = partial(g, n)
} by {
    if forall(i: Nat) { i < n implies f(i) = g(i) } {
        partial_pointwise_eq[R](f, g, n)
        partial(f, n) = partial(g, n)
    }
}

/// A partial sum of negated summands is the negation of the partial sum.
lemma polynomial_partial_neg[R: Ring](f: Nat -> R, n: Nat) {
    partial(polynomial_neg_fn(f), n) = -partial(f, n)
} by {
    define statement(k: Nat) -> Bool {
        partial(polynomial_neg_fn(f), k) = -partial(f, k)
    }

    partial_zero[R](polynomial_neg_fn(f))
    partial(polynomial_neg_fn(f), Nat.0) = R.0
    partial_zero[R](f)
    partial(f, Nat.0) = R.0
    -partial(f, Nat.0) = -R.0
    partial(polynomial_neg_fn(f), Nat.0) = -partial(f, Nat.0)
    statement(Nat.0)

    forall(k: Nat) {
        if statement(k) {
            partial_split_last[R](polynomial_neg_fn(f), k)
            partial(polynomial_neg_fn(f), k.suc) =
                partial(polynomial_neg_fn(f), k) + polynomial_neg_fn(f)(k)
            polynomial_neg_fn(f, k) = -f(k)
            partial_split_last[R](f, k)
            partial(f, k.suc) = partial(f, k) + f(k)
            -partial(f, k.suc) = -(partial(f, k) + f(k))
            inverse_add(partial(f, k), f(k))
            -(partial(f, k) + f(k)) = -f(k) + -partial(f, k)
            -f(k) + -partial(f, k) = -partial(f, k) + -f(k)
            statement(k) = (partial(polynomial_neg_fn(f), k) = -partial(f, k))
            partial(polynomial_neg_fn(f), k) = -partial(f, k)
            partial(polynomial_neg_fn(f), k.suc) = -partial(f, k) + -f(k)
            partial(polynomial_neg_fn(f), k.suc) = -partial(f, k.suc)
            statement(k.suc)
        }
    }

    statement(Nat.0) and forall(k: Nat) { statement(k) implies statement(k.suc) }
    alt_induction(statement)
    forall(k: Nat) { statement(k) }
    statement(n)
}

/// The embedding of the coefficient ring into polynomials preserves addition.
theorem polynomial_constant_add[R: Semiring](r: R, s: R) {
    polynomial_constant(r + s) = polynomial_constant(r) + polynomial_constant(s)
} by {
    forall(k: Nat) {
        if k = Nat.0 {
            polynomial_constant_coeff_zero(r + s)
            polynomial_constant(r + s).coeff(k) = r + s
            polynomial_add_coeff(polynomial_constant(r), polynomial_constant(s), k)
            (polynomial_constant(r) + polynomial_constant(s)).coeff(k) =
                polynomial_constant(r).coeff(k) + polynomial_constant(s).coeff(k)
            polynomial_constant_coeff_zero(r)
            polynomial_constant(r).coeff(k) = r
            polynomial_constant_coeff_zero(s)
            polynomial_constant(s).coeff(k) = s
            (polynomial_constant(r) + polynomial_constant(s)).coeff(k) = r + s
            polynomial_constant(r + s).coeff(k) =
                (polynomial_constant(r) + polynomial_constant(s)).coeff(k)
        } else {
            k != Nat.0
            polynomial_constant_coeff_of_ne_zero(r + s, k)
            polynomial_constant(r + s).coeff(k) = R.0
            polynomial_add_coeff(polynomial_constant(r), polynomial_constant(s), k)
            (polynomial_constant(r) + polynomial_constant(s)).coeff(k) =
                polynomial_constant(r).coeff(k) + polynomial_constant(s).coeff(k)
            polynomial_constant_coeff_of_ne_zero(r, k)
            polynomial_constant(r).coeff(k) = R.0
            polynomial_constant_coeff_of_ne_zero(s, k)
            polynomial_constant(s).coeff(k) = R.0
            (polynomial_constant(r) + polynomial_constant(s)).coeff(k) = R.0 + R.0
            R.0 + R.0 = R.0
            polynomial_constant(r + s).coeff(k) =
                (polynomial_constant(r) + polynomial_constant(s)).coeff(k)
        }
    }
    polynomial_ext_pointwise(polynomial_constant(r + s),
        polynomial_constant(r) + polynomial_constant(s))
}

/// The embedding of the coefficient ring into polynomials preserves
/// multiplication.
theorem polynomial_constant_mul[R: Semiring](r: R, s: R) {
    polynomial_mul(polynomial_constant(r), polynomial_constant(s)) = polynomial_constant(r * s)
} by {
    forall(k: Nat) {
        let term_fn: Nat -> R = function(i: Nat) {
            polynomial_mul_term_coeff(polynomial_constant(r), polynomial_constant(s), k, i)
        }
        if k = Nat.0 {
            polynomial_mul_coeff_apply(polynomial_constant(r), polynomial_constant(s), k)
            polynomial_mul(polynomial_constant(r), polynomial_constant(s)).coeff(k) =
                polynomial_mul_coeff(polynomial_constant(r), polynomial_constant(s), k)
            polynomial_mul_coeff(polynomial_constant(r), polynomial_constant(s), k) =
                partial(function(i: Nat) {
                    polynomial_mul_term_coeff(polynomial_constant(r), polynomial_constant(s), k, i)
                }, k.suc)
            polynomial_mul_coeff(polynomial_constant(r), polynomial_constant(s), k) =
                partial(term_fn, k.suc)
            partial_one[R](term_fn)
            partial(term_fn, Nat.0.suc) = term_fn(Nat.0)
            term_fn(Nat.0) = polynomial_mul_term_coeff(polynomial_constant(r),
                polynomial_constant(s), k, Nat.0)
            polynomial_mul_term_coeff(polynomial_constant(r), polynomial_constant(s), k, Nat.0) =
                polynomial_constant(r).coeff(Nat.0) * polynomial_constant(s).coeff(k - Nat.0)
            polynomial_constant_coeff_zero(r)
            polynomial_constant(r).coeff(Nat.0) = r
            k = Nat.0
            k - Nat.0 = Nat.0 - Nat.0
            Nat.0 - Nat.0 = Nat.0
            polynomial_constant_coeff_zero(s)
            polynomial_constant(s).coeff(Nat.0) = s
            polynomial_constant(s).coeff(k - Nat.0) = s
            polynomial_mul_term_coeff(polynomial_constant(r), polynomial_constant(s), k, Nat.0) =
                r * s
            term_fn(Nat.0) = r * s
            partial(term_fn, k.suc) = r * s
            polynomial_mul(polynomial_constant(r), polynomial_constant(s)).coeff(k) = r * s
            polynomial_constant_coeff_zero(r * s)
            polynomial_constant(r * s).coeff(k) = r * s
            polynomial_mul(polynomial_constant(r), polynomial_constant(s)).coeff(k) =
                polynomial_constant(r * s).coeff(k)
        } else {
            k != Nat.0
            forall(i: Nat) {
                if i < k.suc {
                    if i != Nat.0 {
                        polynomial_constant_coeff_of_ne_zero(r, i)
                        polynomial_constant(r).coeff(i) = R.0
                        term_fn(i) = polynomial_mul_term_coeff(polynomial_constant(r),
                            polynomial_constant(s), k, i)
                        polynomial_mul_term_coeff(polynomial_constant(r), polynomial_constant(s), k, i) =
                            polynomial_constant(r).coeff(i) * polynomial_constant(s).coeff(k - i)
                        polynomial_mul_term_coeff(polynomial_constant(r), polynomial_constant(s), k, i) =
                            R.0 * polynomial_constant(s).coeff(k - i)
                        R.0 * polynomial_constant(s).coeff(k - i) = R.0
                        term_fn(i) = R.0
                    } else {
                        i = Nat.0
                        if k - i = Nat.0 {
                            k = Nat.0
                            false
                        }
                        k - i != Nat.0
                        polynomial_constant_coeff_of_ne_zero(s, k - i)
                        polynomial_constant(s).coeff(k - i) = R.0
                        term_fn(i) = polynomial_mul_term_coeff(polynomial_constant(r),
                            polynomial_constant(s), k, i)
                        polynomial_mul_term_coeff(polynomial_constant(r), polynomial_constant(s), k, i) =
                            polynomial_constant(r).coeff(i) * polynomial_constant(s).coeff(k - i)
                        polynomial_mul_term_coeff(polynomial_constant(r), polynomial_constant(s), k, i) =
                            polynomial_constant(r).coeff(i) * R.0
                        polynomial_constant(r).coeff(i) * R.0 = R.0
                        term_fn(i) = R.0
                    }
                }
            }
            polynomial_partial_zero_of_pointwise_zero(term_fn, k.suc)
            partial(term_fn, k.suc) = R.0
            polynomial_mul_coeff_apply(polynomial_constant(r), polynomial_constant(s), k)
            polynomial_mul(polynomial_constant(r), polynomial_constant(s)).coeff(k) =
                polynomial_mul_coeff(polynomial_constant(r), polynomial_constant(s), k)
            polynomial_mul_coeff(polynomial_constant(r), polynomial_constant(s), k) =
                partial(function(i: Nat) {
                    polynomial_mul_term_coeff(polynomial_constant(r), polynomial_constant(s), k, i)
                }, k.suc)
            polynomial_mul_coeff(polynomial_constant(r), polynomial_constant(s), k) =
                partial(term_fn, k.suc)
            polynomial_mul(polynomial_constant(r), polynomial_constant(s)).coeff(k) = R.0
            polynomial_constant_coeff_of_ne_zero(r * s, k)
            polynomial_constant(r * s).coeff(k) = R.0
            polynomial_mul(polynomial_constant(r), polynomial_constant(s)).coeff(k) =
                polynomial_constant(r * s).coeff(k)
        }
    }
    polynomial_ext_pointwise(polynomial_mul(polynomial_constant(r), polynomial_constant(s)),
        polynomial_constant(r * s))
}

/// The constant polynomial with coefficient one is the one polynomial.
theorem polynomial_constant_one[R: Semiring] {
    polynomial_constant(R.1) = Polynomial[R].one
} by {
    forall(k: Nat) {
        if k = Nat.0 {
            polynomial_constant_coeff_zero(R.1)
            polynomial_constant(R.1).coeff(k) = R.1
            polynomial_one_coeff_zero[R]
            Polynomial[R].one.coeff(k) = R.1
            polynomial_constant(R.1).coeff(k) = Polynomial[R].one.coeff(k)
        } else {
            k != Nat.0
            polynomial_constant_coeff_of_ne_zero(R.1, k)
            polynomial_constant(R.1).coeff(k) = R.0
            polynomial_one_coeff_of_ne_zero[R](k)
            Polynomial[R].one.coeff(k) = R.0
            polynomial_constant(R.1).coeff(k) = Polynomial[R].one.coeff(k)
        }
    }
    polynomial_ext_pointwise(polynomial_constant(R.1), Polynomial[R].one)
}

/// The constant polynomial with coefficient zero is the zero polynomial.
theorem polynomial_constant_zero_poly[R: Semiring] {
    polynomial_constant(R.0) = Polynomial[R].zero
} by {
    forall(k: Nat) {
        if k = Nat.0 {
            polynomial_constant_coeff_zero(R.0)
            polynomial_constant(R.0).coeff(k) = R.0
            polynomial_zero_coeff[R](k)
            Polynomial[R].zero.coeff(k) = R.0
            polynomial_constant(R.0).coeff(k) = Polynomial[R].zero.coeff(k)
        } else {
            k != Nat.0
            polynomial_constant_coeff_of_ne_zero(R.0, k)
            polynomial_constant(R.0).coeff(k) = R.0
            polynomial_zero_coeff[R](k)
            Polynomial[R].zero.coeff(k) = R.0
            polynomial_constant(R.0).coeff(k) = Polynomial[R].zero.coeff(k)
        }
    }
    polynomial_ext_pointwise(polynomial_constant(R.0), Polynomial[R].zero)
}

/// A constant polynomial is the zero polynomial exactly when its coefficient
/// is zero.
theorem polynomial_constant_eq_zero[R: Semiring](r: R) {
    (polynomial_constant(r) = Polynomial[R].zero) = (r = R.0)
} by {
    if polynomial_constant(r) = Polynomial[R].zero {
        polynomial_eq_coeff_at(polynomial_constant(r), Polynomial[R].zero, Nat.0)
        polynomial_constant(r).coeff(Nat.0) = Polynomial[R].zero.coeff(Nat.0)
        polynomial_constant_coeff_zero(r)
        polynomial_constant(r).coeff(Nat.0) = r
        polynomial_zero_coeff[R](Nat.0)
        Polynomial[R].zero.coeff(Nat.0) = R.0
        r = R.0
    }
    if r = R.0 {
        polynomial_constant(r) = polynomial_constant(R.0)
        polynomial_constant_zero_poly[R]
        polynomial_constant(R.0) = Polynomial[R].zero
        polynomial_constant(r) = Polynomial[R].zero
    }
}

/// The constant-polynomial embedding is injective.
theorem polynomial_constant_injective[R: Semiring](r: R, s: R) {
    polynomial_constant(r) = polynomial_constant(s) implies r = s
} by {
    if polynomial_constant(r) = polynomial_constant(s) {
        polynomial_eq_coeff_at(polynomial_constant(r), polynomial_constant(s), Nat.0)
        polynomial_constant(r).coeff(Nat.0) = polynomial_constant(s).coeff(Nat.0)
        polynomial_constant_coeff_zero(r)
        polynomial_constant(r).coeff(Nat.0) = r
        polynomial_constant_coeff_zero(s)
        polynomial_constant(s).coeff(Nat.0) = s
        r = s
    }
}

/// A constant polynomial with nonzero coefficient is not the zero polynomial.
theorem polynomial_constant_ne_zero[R: Semiring](r: R) {
    r != R.0 implies polynomial_constant(r) != Polynomial[R].zero
} by {
    if r != R.0 {
        if polynomial_constant(r) = Polynomial[R].zero {
            polynomial_constant_eq_zero(r)
            r = R.0
            false
        }
        polynomial_constant(r) != Polynomial[R].zero
    }
}

/// Polynomial multiplication distributes over addition on the right.
theorem polynomial_mul_add_right[R: CommRing](
    p: Polynomial[R], q: Polynomial[R], r: Polynomial[R]
) {
    polynomial_mul(p + q, r) = polynomial_mul(p, r) + polynomial_mul(q, r)
} by {
    polynomial_mul_comm[R](p + q, r)
    polynomial_mul(p + q, r) = polynomial_mul(r, p + q)
    polynomial_mul_add_left[R](r, p, q)
    polynomial_mul(r, p + q) = polynomial_mul(r, p) + polynomial_mul(r, q)
    polynomial_mul_comm[R](r, p)
    polynomial_mul(r, p) = polynomial_mul(p, r)
    polynomial_mul_comm[R](r, q)
    polynomial_mul(r, q) = polynomial_mul(q, r)
    polynomial_mul(r, p) + polynomial_mul(r, q) =
        polynomial_mul(p, r) + polynomial_mul(q, r)
    polynomial_mul(p + q, r) = polynomial_mul(p, r) + polynomial_mul(q, r)
}

/// Negation on the right factor negates the whole product.
theorem polynomial_mul_neg_right[R: Ring](p: Polynomial[R], q: Polynomial[R]) {
    polynomial_mul(p, q.neg) = polynomial_mul(p, q).neg
} by {
    forall(k: Nat) {
        let term_fn: Nat -> R = function(i: Nat) {
            polynomial_mul_term_coeff(p, q.neg, k, i)
        }
        let neg_fn: Nat -> R = function(i: Nat) {
            polynomial_neg_fn(polynomial_mul_term_coeff(p, q, k), i)
        }
        forall(i: Nat) {
            if i < k.suc {
                term_fn(i) = polynomial_mul_term_coeff(p, q.neg, k, i)
                polynomial_mul_term_coeff(p, q.neg, k, i) =
                    p.coeff(i) * q.neg.coeff(k - i)
                polynomial_neg_coeff(q, k - i)
                q.neg.coeff(k - i) = -q.coeff(k - i)
                polynomial_mul_term_coeff(p, q.neg, k, i) = p.coeff(i) * -q.coeff(k - i)
                mul_neg_right(p.coeff(i), q.coeff(k - i))
                p.coeff(i) * -q.coeff(k - i) = -(p.coeff(i) * q.coeff(k - i))
                polynomial_mul_term_coeff(p, q, k, i) = p.coeff(i) * q.coeff(k - i)
                neg_fn(i) = polynomial_neg_fn(polynomial_mul_term_coeff(p, q, k), i)
                polynomial_neg_fn(polynomial_mul_term_coeff(p, q, k), i) =
                    -polynomial_mul_term_coeff(p, q, k, i)
                term_fn(i) = neg_fn(i)
            }
        }
        polynomial_function_extensionality_nat(term_fn, neg_fn, k.suc)
        partial(term_fn, k.suc) = partial(neg_fn, k.suc)
        polynomial_mul_coeff_apply(p, q.neg, k)
        polynomial_mul(p, q.neg).coeff(k) = polynomial_mul_coeff(p, q.neg, k)
        polynomial_mul_coeff(p, q.neg, k) =
            partial(function(i: Nat) { polynomial_mul_term_coeff(p, q.neg, k, i) }, k.suc)
        polynomial_mul_coeff(p, q.neg, k) = partial(term_fn, k.suc)
        polynomial_mul_coeff_apply(p, q, k)
        polynomial_mul(p, q).coeff(k) = polynomial_mul_coeff(p, q, k)
        polynomial_neg_coeff(polynomial_mul(p, q), k)
        polynomial_mul(p, q).neg.coeff(k) = -polynomial_mul(p, q).coeff(k)
        polynomial_mul(p, q).neg.coeff(k) = -polynomial_mul_coeff(p, q, k)
        polynomial_partial_neg(polynomial_mul_term_coeff(p, q, k), k.suc)
        partial(polynomial_neg_fn(polynomial_mul_term_coeff(p, q, k)), k.suc) =
            -partial(polynomial_mul_term_coeff(p, q, k), k.suc)
        polynomial_mul_coeff(p, q, k) =
            partial(function(i: Nat) { polynomial_mul_term_coeff(p, q, k, i) }, k.suc)
        polynomial_mul_coeff(p, q, k) = partial(polynomial_mul_term_coeff(p, q, k), k.suc)
        polynomial_function_extensionality_nat(neg_fn,
            polynomial_neg_fn(polynomial_mul_term_coeff(p, q, k)), k.suc)
        partial(neg_fn, k.suc) =
            partial(polynomial_neg_fn(polynomial_mul_term_coeff(p, q, k)), k.suc)
        polynomial_mul(p, q.neg).coeff(k) = polynomial_mul(p, q).neg.coeff(k)
    }
    polynomial_ext_pointwise(polynomial_mul(p, q.neg), polynomial_mul(p, q).neg)
}

/// Negation on the left factor negates the whole product.
theorem polynomial_mul_neg_left[R: CommRing](p: Polynomial[R], q: Polynomial[R]) {
    polynomial_mul(p.neg, q) = polynomial_mul(p, q).neg
} by {
    polynomial_mul_comm[R](p.neg, q)
    polynomial_mul(p.neg, q) = polynomial_mul(q, p.neg)
    polynomial_mul_neg_right(q, p)
    polynomial_mul(q, p.neg) = polynomial_mul(q, p).neg
    polynomial_mul_comm[R](q, p)
    polynomial_mul(q, p) = polynomial_mul(p, q)
    polynomial_mul(q, p).neg = polynomial_mul(p, q).neg
    polynomial_mul(p.neg, q) = polynomial_mul(p, q).neg
}

/// The coefficient of `X^k` in the scalar multiple `r * p` is `r` times the
/// coefficient of `X^k` in `p`.
theorem polynomial_scalar_mul_coeff[R: Semiring](r: R, p: Polynomial[R], k: Nat) {
    polynomial_mul(polynomial_constant(r), p).coeff(k) = r * p.coeff(k)
} by {
    let term_fn: Nat -> R = function(i: Nat) {
        polynomial_mul_term_coeff(polynomial_constant(r), p, k, i)
    }
    forall(i: Nat) {
        if i < k.suc {
            if i != Nat.0 {
                term_fn(i) = polynomial_mul_term_coeff(polynomial_constant(r), p, k, i)
                polynomial_mul_term_coeff(polynomial_constant(r), p, k, i) =
                    polynomial_constant(r).coeff(i) * p.coeff(k - i)
                polynomial_constant_coeff_of_ne_zero(r, i)
                polynomial_constant(r).coeff(i) = R.0
                polynomial_mul_term_coeff(polynomial_constant(r), p, k, i) = R.0 * p.coeff(k - i)
                R.0 * p.coeff(k - i) = R.0
                term_fn(i) = R.0
            }
        }
    }
    polynomial_partial_first_only(term_fn, k)
    partial(term_fn, k.suc) = term_fn(Nat.0)
    term_fn(Nat.0) = polynomial_mul_term_coeff(polynomial_constant(r), p, k, Nat.0)
    polynomial_mul_term_coeff(polynomial_constant(r), p, k, Nat.0) =
        polynomial_constant(r).coeff(Nat.0) * p.coeff(k - Nat.0)
    polynomial_constant_coeff_zero(r)
    polynomial_constant(r).coeff(Nat.0) = r
    k - Nat.0 = k
    polynomial_mul_term_coeff(polynomial_constant(r), p, k, Nat.0) = r * p.coeff(k)
    term_fn(Nat.0) = r * p.coeff(k)
    partial(term_fn, k.suc) = r * p.coeff(k)
    polynomial_mul_coeff_apply(polynomial_constant(r), p, k)
    polynomial_mul(polynomial_constant(r), p).coeff(k) =
        polynomial_mul_coeff(polynomial_constant(r), p, k)
    polynomial_mul_coeff(polynomial_constant(r), p, k) =
        partial(function(i: Nat) { polynomial_mul_term_coeff(polynomial_constant(r), p, k, i) }, k.suc)
    polynomial_mul_coeff(polynomial_constant(r), p, k) = partial(term_fn, k.suc)
    polynomial_mul(polynomial_constant(r), p).coeff(k) = r * p.coeff(k)
}

/// Scalar multiplication distributes over the scalar: `(r + s) * p =
/// r * p + s * p`.
theorem polynomial_scalar_distrib[R: CommRing](r: R, s: R, p: Polynomial[R]) {
    polynomial_mul(polynomial_constant(r + s), p) =
        polynomial_mul(polynomial_constant(r), p) + polynomial_mul(polynomial_constant(s), p)
} by {
    polynomial_constant_add(r, s)
    polynomial_constant(r + s) = polynomial_constant(r) + polynomial_constant(s)
    polynomial_mul_add_left[R](polynomial_constant(r), polynomial_constant(s), p)
    polynomial_mul(polynomial_constant(r) + polynomial_constant(s), p) =
        polynomial_mul(polynomial_constant(r), p) + polynomial_mul(polynomial_constant(s), p)
    polynomial_mul(polynomial_constant(r + s), p) =
        polynomial_mul(polynomial_constant(r), p) + polynomial_mul(polynomial_constant(s), p)
}

/// Polynomial multiplication is associative (coefficient form).
theorem polynomial_mul_assoc_coeff[R: CommRing](
    p: Polynomial[R], q: Polynomial[R], r: Polynomial[R]
) {
    polynomial_mul(polynomial_mul(p, q), r) = polynomial_mul(p, polynomial_mul(q, r))
} by {
    polynomial_mul_assoc[R](p, q, r)
}

/// Polynomial multiplication is commutative (coefficient form).
theorem polynomial_mul_comm_coeff[R: CommRing](p: Polynomial[R], q: Polynomial[R]) {
    polynomial_mul(p, q) = polynomial_mul(q, p)
} by {
    polynomial_mul_comm(p, q)
}
