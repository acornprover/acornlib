/// Vieta-style factorization for linear and quadratic polynomials: the
/// product of the linear factors `X - a` and `X - b` is the monic quadratic
/// `X^2 - (a + b) X + a b`, whose roots are exactly `a` and `b`.

from nat import Nat
from semiring import Semiring
from comm_ring import CommRing
from algebra.add_group import inverse_add
from polynomial import Polynomial, polynomial_constant, polynomial_monomial,
    polynomial_monomial_coeff_self, polynomial_monomial_coeff_of_ne,
    polynomial_constant_coeff_zero, polynomial_constant_coeff_of_ne_zero,
    polynomial_ext_pointwise, polynomial_add_coeff, polynomial_add_assoc,
    polynomial_eval, polynomial_eval_constant,
    polynomial_eval_add, polynomial_eval_mul, polynomial_mul,
    polynomial_x_minus_a_polynomial_witness, polynomial_x_minus_a_polynomial_witness_at,
    polynomial_divisible_by_x_minus_a_polynomial
from polynomial.polynomial_eval_deep import polynomial_eval_monomial
from polynomial.polynomial_monomial import polynomial_monomial_mul,
    polynomial_constant_mul_monomial, polynomial_monomial_mul_constant
from polynomial.polynomial_ring_theory import polynomial_constant_mul, polynomial_mul_add_right
from polynomial_mul_distrib import polynomial_mul_add_left

numerals Nat

/// The linear polynomial `a * X + b`.
define linear_polynomial[R: Semiring](a: R, b: R) -> Polynomial[R] {
    polynomial_constant(b) + polynomial_monomial(Nat.1, a)
}

/// The quadratic polynomial `a * X^2 + b * X + c`.
define quadratic_polynomial[R: Semiring](a: R, b: R, c: R) -> Polynomial[R] {
    polynomial_constant(c) + polynomial_monomial(Nat.1, b) + polynomial_monomial(Nat.2, a)
}

/// The first power of an element is itself.
lemma vieta_pow_one[R: Semiring](x: R) {
    x.pow(Nat.1) = x
} by {
    x.pow(Nat.1) = x.pow(Nat.0.suc)
    x.pow(Nat.0.suc) = x * x.pow(Nat.0)
    x.pow(Nat.0) = R.1
    x.pow(Nat.1) = x * R.1
    x * R.1 = x
}

/// The second power of an element is its square.
lemma vieta_pow_two[R: Semiring](x: R) {
    x.pow(Nat.2) = x * x
} by {
    x.pow(Nat.2) = x.pow(Nat.1.suc)
    x.pow(Nat.1.suc) = x * x.pow(Nat.1)
    vieta_pow_one(x)
    x.pow(Nat.2) = x * x
}

/// Sums of monomials of the same exponent are monomials.
theorem polynomial_monomial_add_same[R: Semiring](n: Nat, r: R, s: R) {
    polynomial_monomial(n, r) + polynomial_monomial(n, s) = polynomial_monomial(n, r + s)
} by {
    forall(k: Nat) {
        if k = n {
            polynomial_add_coeff(polynomial_monomial(n, r), polynomial_monomial(n, s), k)
            (polynomial_monomial(n, r) + polynomial_monomial(n, s)).coeff(k) =
                polynomial_monomial(n, r).coeff(k) + polynomial_monomial(n, s).coeff(k)
            Polynomial[R].monomial(n, r) = polynomial_monomial(n, r)
            polynomial_monomial_coeff_self(n, r)
            polynomial_monomial(n, r).coeff(k) = r
            Polynomial[R].monomial(n, s) = polynomial_monomial(n, s)
            polynomial_monomial_coeff_self(n, s)
            polynomial_monomial(n, s).coeff(k) = s
            (polynomial_monomial(n, r) + polynomial_monomial(n, s)).coeff(k) = r + s
            polynomial_monomial(n, r + s).coeff(k) = r + s
            (polynomial_monomial(n, r) + polynomial_monomial(n, s)).coeff(k) =
                polynomial_monomial(n, r + s).coeff(k)
        } else {
            k != n
            polynomial_add_coeff(polynomial_monomial(n, r), polynomial_monomial(n, s), k)
            (polynomial_monomial(n, r) + polynomial_monomial(n, s)).coeff(k) =
                polynomial_monomial(n, r).coeff(k) + polynomial_monomial(n, s).coeff(k)
            Polynomial[R].monomial(n, r) = polynomial_monomial(n, r)
            polynomial_monomial_coeff_of_ne[R](n, r, k)
            polynomial_monomial(n, r).coeff(k) = R.0
            Polynomial[R].monomial(n, s) = polynomial_monomial(n, s)
            polynomial_monomial_coeff_of_ne[R](n, s, k)
            polynomial_monomial(n, s).coeff(k) = R.0
            (polynomial_monomial(n, r) + polynomial_monomial(n, s)).coeff(k) = R.0 + R.0
            R.0 + R.0 = R.0
            polynomial_monomial(n, r + s).coeff(k) = R.0
            (polynomial_monomial(n, r) + polynomial_monomial(n, s)).coeff(k) =
                polynomial_monomial(n, r + s).coeff(k)
        }
    }
    polynomial_ext_pointwise(polynomial_monomial(n, r) + polynomial_monomial(n, s),
        polynomial_monomial(n, r + s))
}

/// The constant coefficient of a linear polynomial.
theorem linear_polynomial_coeff_zero[R: Semiring](a: R, b: R) {
    linear_polynomial(a, b).coeff(Nat.0) = b
} by {
    polynomial_add_coeff(polynomial_constant(b), polynomial_monomial(Nat.1, a), Nat.0)
    linear_polynomial(a, b).coeff(Nat.0) =
        polynomial_constant(b).coeff(Nat.0) + polynomial_monomial(Nat.1, a).coeff(Nat.0)
    polynomial_constant_coeff_zero(b)
    polynomial_constant(b).coeff(Nat.0) = b
    Nat.0 != Nat.1
    polynomial_monomial_coeff_of_ne(Nat.1, a, Nat.0)
    Polynomial[R].monomial(Nat.1, a) = polynomial_monomial(Nat.1, a)
    polynomial_monomial(Nat.1, a).coeff(Nat.0) = R.0
    linear_polynomial(a, b).coeff(Nat.0) = b + R.0
    b + R.0 = b
}

/// The linear coefficient of a linear polynomial.
theorem linear_polynomial_coeff_one[R: Semiring](a: R, b: R) {
    linear_polynomial(a, b).coeff(Nat.1) = a
} by {
    polynomial_add_coeff(polynomial_constant(b), polynomial_monomial(Nat.1, a), Nat.1)
    linear_polynomial(a, b).coeff(Nat.1) =
        polynomial_constant(b).coeff(Nat.1) + polynomial_monomial(Nat.1, a).coeff(Nat.1)
    Nat.1 != Nat.0
    polynomial_constant_coeff_of_ne_zero(b, Nat.1)
    polynomial_constant(b).coeff(Nat.1) = R.0
    polynomial_monomial_coeff_self(Nat.1, a)
    Polynomial[R].monomial(Nat.1, a) = polynomial_monomial(Nat.1, a)
    polynomial_monomial(Nat.1, a).coeff(Nat.1) = a
    linear_polynomial(a, b).coeff(Nat.1) = R.0 + a
    R.0 + a = a
}

/// The value of a linear polynomial at a point.
theorem linear_polynomial_eval[R: CommRing](a: R, b: R, x: R) {
    polynomial_eval(linear_polynomial(a, b), x) = b + a * x
} by {
    polynomial_eval_add(polynomial_constant(b), polynomial_monomial(Nat.1, a), x)
    polynomial_eval(linear_polynomial(a, b), x) =
        polynomial_eval(polynomial_constant(b), x) + polynomial_eval(polynomial_monomial(Nat.1, a), x)
    polynomial_eval_constant(b, x)
    polynomial_eval(polynomial_constant(b), x) = b
    polynomial_eval_monomial(Nat.1, a, x)
    polynomial_eval(polynomial_monomial(Nat.1, a), x) = a * x.pow(Nat.1)
    vieta_pow_one(x)
    polynomial_eval(polynomial_monomial(Nat.1, a), x) = a * x
    polynomial_eval(linear_polynomial(a, b), x) = b + a * x
}

/// The value of a quadratic polynomial at a point.
theorem quadratic_polynomial_eval[R: CommRing](a: R, b: R, c: R, x: R) {
    polynomial_eval(quadratic_polynomial(a, b, c), x) = c + b * x + a * x * x
} by {
    polynomial_eval_add(polynomial_constant(c) + polynomial_monomial(Nat.1, b),
        polynomial_monomial(Nat.2, a), x)
    polynomial_eval(quadratic_polynomial(a, b, c), x) =
        polynomial_eval(polynomial_constant(c) + polynomial_monomial(Nat.1, b), x) +
        polynomial_eval(polynomial_monomial(Nat.2, a), x)
    polynomial_eval_add(polynomial_constant(c), polynomial_monomial(Nat.1, b), x)
    polynomial_eval(polynomial_constant(c) + polynomial_monomial(Nat.1, b), x) =
        polynomial_eval(polynomial_constant(c), x) + polynomial_eval(polynomial_monomial(Nat.1, b), x)
    polynomial_eval_constant(c, x)
    polynomial_eval(polynomial_constant(c), x) = c
    polynomial_eval_monomial(Nat.1, b, x)
    polynomial_eval(polynomial_monomial(Nat.1, b), x) = b * x.pow(Nat.1)
    vieta_pow_one(x)
    polynomial_eval(polynomial_monomial(Nat.1, b), x) = b * x
    polynomial_eval_monomial(Nat.2, a, x)
    polynomial_eval(polynomial_monomial(Nat.2, a), x) = a * x.pow(Nat.2)
    vieta_pow_two(x)
    polynomial_eval(polynomial_monomial(Nat.2, a), x) = a * (x * x)
    polynomial_eval(polynomial_constant(c) + polynomial_monomial(Nat.1, b), x) = c + b * x
    polynomial_eval(quadratic_polynomial(a, b, c), x) = c + b * x + a * (x * x)
    a * (x * x) = a * x * x
    polynomial_eval(quadratic_polynomial(a, b, c), x) = c + b * x + a * x * x
}

/// The polynomial `X - a` has `a` as a root.
theorem linear_polynomial_neg_root[R: CommRing](a: R) {
    polynomial_eval(linear_polynomial(R.1, -a), a) = R.0
} by {
    linear_polynomial_eval(R.1, -a, a)
    polynomial_eval(linear_polynomial(R.1, -a), a) = -a + R.1 * a
    R.1 * a = a
    polynomial_eval(linear_polynomial(R.1, -a), a) = -a + a
    -a + a = a + -a
    a + -a = R.0
    polynomial_eval(linear_polynomial(R.1, -a), a) = R.0
}

/// A root of the first factor is a root of the product.
theorem polynomial_root_mul_left[R: CommRing](p: Polynomial[R], q: Polynomial[R], a: R) {
    polynomial_eval(p, a) = R.0 implies polynomial_eval(polynomial_mul(p, q), a) = R.0
} by {
    if polynomial_eval(p, a) = R.0 {
        polynomial_eval_mul(p, q, a)
        polynomial_eval(polynomial_mul(p, q), a) = polynomial_eval(p, a) * polynomial_eval(q, a)
        polynomial_eval(p, a) = R.0
        polynomial_eval(polynomial_mul(p, q), a) = R.0 * polynomial_eval(q, a)
        R.0 * polynomial_eval(q, a) = R.0
        polynomial_eval(polynomial_mul(p, q), a) = R.0
    }
}

/// A root of the second factor is a root of the product.
theorem polynomial_root_mul_right[R: CommRing](p: Polynomial[R], q: Polynomial[R], a: R) {
    polynomial_eval(q, a) = R.0 implies polynomial_eval(polynomial_mul(p, q), a) = R.0
} by {
    if polynomial_eval(q, a) = R.0 {
        polynomial_eval_mul(p, q, a)
        polynomial_eval(polynomial_mul(p, q), a) = polynomial_eval(p, a) * polynomial_eval(q, a)
        polynomial_eval(q, a) = R.0
        polynomial_eval(polynomial_mul(p, q), a) = polynomial_eval(p, a) * R.0
        polynomial_eval(p, a) * R.0 = R.0
        polynomial_eval(polynomial_mul(p, q), a) = R.0
    }
}

/// The product of two linear polynomials expands into four monomial terms.
lemma linear_product_expansion[R: CommRing](
    a: R, b: R, c: R, d: R
) {
    polynomial_mul(linear_polynomial(a, b), linear_polynomial(c, d)) =
        polynomial_constant(b * d) + polynomial_monomial(Nat.1, b * c) +
        polynomial_monomial(Nat.1, a * d) + polynomial_monomial(Nat.2, a * c)
} by {
    polynomial_mul_add_left(polynomial_constant(b), polynomial_monomial(Nat.1, a),
        linear_polynomial(c, d))
    polynomial_mul(linear_polynomial(a, b), linear_polynomial(c, d)) =
        polynomial_mul(polynomial_constant(b), linear_polynomial(c, d)) +
        polynomial_mul(polynomial_monomial(Nat.1, a), linear_polynomial(c, d))
    polynomial_mul_add_left(polynomial_constant(b), polynomial_constant(d), polynomial_monomial(Nat.1, c))
    polynomial_mul(polynomial_constant(b), polynomial_constant(d) + polynomial_monomial(Nat.1, c)) =
        polynomial_mul(polynomial_constant(b), polynomial_constant(d)) +
        polynomial_mul(polynomial_constant(b), polynomial_monomial(Nat.1, c))
    polynomial_mul_add_left(polynomial_monomial(Nat.1, a), polynomial_constant(d), polynomial_monomial(Nat.1, c))
    polynomial_mul(polynomial_monomial(Nat.1, a), polynomial_constant(d) + polynomial_monomial(Nat.1, c)) =
        polynomial_mul(polynomial_monomial(Nat.1, a), polynomial_constant(d)) +
        polynomial_mul(polynomial_monomial(Nat.1, a), polynomial_monomial(Nat.1, c))
    polynomial_mul(polynomial_constant(b), linear_polynomial(c, d)) =
        polynomial_mul(polynomial_constant(b), polynomial_constant(d)) +
        polynomial_mul(polynomial_constant(b), polynomial_monomial(Nat.1, c))
    polynomial_mul(polynomial_monomial(Nat.1, a), linear_polynomial(c, d)) =
        polynomial_mul(polynomial_monomial(Nat.1, a), polynomial_constant(d)) +
        polynomial_mul(polynomial_monomial(Nat.1, a), polynomial_monomial(Nat.1, c))
    polynomial_constant_mul(b, d)
    polynomial_mul(polynomial_constant(b), polynomial_constant(d)) = polynomial_constant(b * d)
    polynomial_constant_mul_monomial(b, Nat.1, c)
    polynomial_mul(polynomial_constant(b), polynomial_monomial(Nat.1, c)) =
        polynomial_monomial(Nat.1, b * c)
    polynomial_monomial_mul_constant(Nat.1, a, d)
    polynomial_mul(polynomial_monomial(Nat.1, a), polynomial_constant(d)) =
        polynomial_monomial(Nat.1, a * d)
    polynomial_monomial_mul(Nat.1, a, Nat.1, c)
    polynomial_mul(polynomial_monomial(Nat.1, a), polynomial_monomial(Nat.1, c)) =
        polynomial_monomial(Nat.1 + Nat.1, a * c)
    Nat.1 + Nat.1 = Nat.2
    polynomial_mul(polynomial_monomial(Nat.1, a), polynomial_monomial(Nat.1, c)) =
        polynomial_monomial(Nat.2, a * c)
    polynomial_mul(linear_polynomial(a, b), linear_polynomial(c, d)) =
        (polynomial_constant(b * d) + polynomial_monomial(Nat.1, b * c)) +
        (polynomial_monomial(Nat.1, a * d) + polynomial_monomial(Nat.2, a * c))
    polynomial_add_assoc(polynomial_constant(b * d) + polynomial_monomial(Nat.1, b * c),
        polynomial_monomial(Nat.1, a * d), polynomial_monomial(Nat.2, a * c))
    ((polynomial_constant(b * d) + polynomial_monomial(Nat.1, b * c)) +
        polynomial_monomial(Nat.1, a * d)) + polynomial_monomial(Nat.2, a * c) =
        (polynomial_constant(b * d) + polynomial_monomial(Nat.1, b * c)) +
        (polynomial_monomial(Nat.1, a * d) + polynomial_monomial(Nat.2, a * c))
    polynomial_mul(linear_polynomial(a, b), linear_polynomial(c, d)) =
        (polynomial_constant(b * d) + polynomial_monomial(Nat.1, b * c)) +
        polynomial_monomial(Nat.1, a * d) + polynomial_monomial(Nat.2, a * c)
}

/// The product of the monic linear factors `X - a` and `X - b` is the monic
/// quadratic `X^2 - (a + b) X + a b`.
theorem polynomial_linear_mul_quadratic[R: CommRing](a: R, b: R) {
    polynomial_mul(linear_polynomial(R.1, -a), linear_polynomial(R.1, -b)) =
        quadratic_polynomial(R.1, -(a + b), a * b)
} by {
    linear_product_expansion(R.1, -a, R.1, -b)
    polynomial_mul(linear_polynomial(R.1, -a), linear_polynomial(R.1, -b)) =
        polynomial_constant((-a) * (-b)) + polynomial_monomial(Nat.1, (-a) * R.1) +
        polynomial_monomial(Nat.1, R.1 * (-b)) + polynomial_monomial(Nat.2, R.1 * R.1)
    (-a) * (-b) = a * b
    polynomial_constant((-a) * (-b)) = polynomial_constant(a * b)
    (-a) * R.1 = -a
    polynomial_monomial(Nat.1, (-a) * R.1) = polynomial_monomial(Nat.1, -a)
    R.1 * (-b) = -b
    polynomial_monomial(Nat.1, R.1 * (-b)) = polynomial_monomial(Nat.1, -b)
    R.1 * R.1 = R.1
    polynomial_monomial(Nat.2, R.1 * R.1) = polynomial_monomial(Nat.2, R.1)
    polynomial_mul(linear_polynomial(R.1, -a), linear_polynomial(R.1, -b)) =
        polynomial_constant(a * b) + polynomial_monomial(Nat.1, -a) +
        polynomial_monomial(Nat.1, -b) + polynomial_monomial(Nat.2, R.1)
    polynomial_add_assoc(polynomial_constant(a * b) + polynomial_monomial(Nat.1, -a),
        polynomial_monomial(Nat.1, -b), polynomial_monomial(Nat.2, R.1))
    (polynomial_constant(a * b) + polynomial_monomial(Nat.1, -a) +
        polynomial_monomial(Nat.1, -b)) + polynomial_monomial(Nat.2, R.1) =
        (polynomial_constant(a * b) + polynomial_monomial(Nat.1, -a)) +
        (polynomial_monomial(Nat.1, -b) + polynomial_monomial(Nat.2, R.1))
    polynomial_add_assoc(polynomial_constant(a * b), polynomial_monomial(Nat.1, -a),
        polynomial_monomial(Nat.1, -b) + polynomial_monomial(Nat.2, R.1))
    (polynomial_constant(a * b) + polynomial_monomial(Nat.1, -a)) +
        (polynomial_monomial(Nat.1, -b) + polynomial_monomial(Nat.2, R.1)) =
        polynomial_constant(a * b) + (polynomial_monomial(Nat.1, -a) +
            (polynomial_monomial(Nat.1, -b) + polynomial_monomial(Nat.2, R.1)))
    polynomial_add_assoc(polynomial_monomial(Nat.1, -a), polynomial_monomial(Nat.1, -b),
        polynomial_monomial(Nat.2, R.1))
    polynomial_monomial(Nat.1, -a) + (polynomial_monomial(Nat.1, -b) + polynomial_monomial(Nat.2, R.1)) =
        (polynomial_monomial(Nat.1, -a) + polynomial_monomial(Nat.1, -b)) +
        polynomial_monomial(Nat.2, R.1)
    polynomial_constant(a * b) + (polynomial_monomial(Nat.1, -a) +
        (polynomial_monomial(Nat.1, -b) + polynomial_monomial(Nat.2, R.1))) =
        polynomial_constant(a * b) + ((polynomial_monomial(Nat.1, -a) +
            polynomial_monomial(Nat.1, -b)) + polynomial_monomial(Nat.2, R.1))
    polynomial_add_assoc(polynomial_constant(a * b),
        polynomial_monomial(Nat.1, -a) + polynomial_monomial(Nat.1, -b),
        polynomial_monomial(Nat.2, R.1))
    polynomial_constant(a * b) + ((polynomial_monomial(Nat.1, -a) +
        polynomial_monomial(Nat.1, -b)) + polynomial_monomial(Nat.2, R.1)) =
        (polynomial_constant(a * b) + (polynomial_monomial(Nat.1, -a) +
            polynomial_monomial(Nat.1, -b))) + polynomial_monomial(Nat.2, R.1)
    polynomial_monomial_add_same(Nat.1, -a, -b)
    polynomial_monomial(Nat.1, -a) + polynomial_monomial(Nat.1, -b) =
        polynomial_monomial(Nat.1, -a + -b)
    inverse_add(a, b)
    -(a + b) = -b + -a
    -b + -a = -a + -b
    -a + -b = -(a + b)
    polynomial_monomial(Nat.1, -a + -b) = polynomial_monomial(Nat.1, -(a + b))
    polynomial_monomial(Nat.1, -a) + polynomial_monomial(Nat.1, -b) =
        polynomial_monomial(Nat.1, -(a + b))
    (polynomial_constant(a * b) + (polynomial_monomial(Nat.1, -a) +
        polynomial_monomial(Nat.1, -b))) + polynomial_monomial(Nat.2, R.1) =
        (polynomial_constant(a * b) + polynomial_monomial(Nat.1, -(a + b))) +
        polynomial_monomial(Nat.2, R.1)
    polynomial_mul(linear_polynomial(R.1, -a), linear_polynomial(R.1, -b)) =
        (polynomial_constant(a * b) + polynomial_monomial(Nat.1, -(a + b))) +
        polynomial_monomial(Nat.2, R.1)
    quadratic_polynomial(R.1, -(a + b), a * b) =
        polynomial_constant(a * b) + polynomial_monomial(Nat.1, -(a + b)) +
        polynomial_monomial(Nat.2, R.1)
    quadratic_polynomial(R.1, -(a + b), a * b) =
        (polynomial_constant(a * b) + polynomial_monomial(Nat.1, -(a + b))) +
        polynomial_monomial(Nat.2, R.1)
    polynomial_mul(linear_polynomial(R.1, -a), linear_polynomial(R.1, -b)) =
        quadratic_polynomial(R.1, -(a + b), a * b)
}

/// The value of the monic quadratic with roots `a` and `b` at `a` is zero.
theorem quadratic_root_vieta_a[R: CommRing](a: R, b: R) {
    polynomial_eval(quadratic_polynomial(R.1, -(a + b), a * b), a) = R.0
} by {
    polynomial_linear_mul_quadratic(a, b)
    polynomial_mul(linear_polynomial(R.1, -a), linear_polynomial(R.1, -b)) =
        quadratic_polynomial(R.1, -(a + b), a * b)
    polynomial_eval(quadratic_polynomial(R.1, -(a + b), a * b), a) =
        polynomial_eval(polynomial_mul(linear_polynomial(R.1, -a), linear_polynomial(R.1, -b)), a)
    linear_polynomial_neg_root(a)
    polynomial_eval(linear_polynomial(R.1, -a), a) = R.0
    polynomial_root_mul_left(linear_polynomial(R.1, -a), linear_polynomial(R.1, -b), a)
    polynomial_eval(polynomial_mul(linear_polynomial(R.1, -a), linear_polynomial(R.1, -b)), a) = R.0
    polynomial_eval(quadratic_polynomial(R.1, -(a + b), a * b), a) = R.0
}

/// The value of the monic quadratic with roots `a` and `b` at `b` is zero.
theorem quadratic_root_vieta_b[R: CommRing](a: R, b: R) {
    polynomial_eval(quadratic_polynomial(R.1, -(a + b), a * b), b) = R.0
} by {
    polynomial_linear_mul_quadratic(a, b)
    polynomial_mul(linear_polynomial(R.1, -a), linear_polynomial(R.1, -b)) =
        quadratic_polynomial(R.1, -(a + b), a * b)
    polynomial_eval(quadratic_polynomial(R.1, -(a + b), a * b), b) =
        polynomial_eval(polynomial_mul(linear_polynomial(R.1, -a), linear_polynomial(R.1, -b)), b)
    linear_polynomial_neg_root(b)
    polynomial_eval(linear_polynomial(R.1, -b), b) = R.0
    polynomial_root_mul_right(linear_polynomial(R.1, -a), linear_polynomial(R.1, -b), b)
    polynomial_eval(polynomial_mul(linear_polynomial(R.1, -a), linear_polynomial(R.1, -b)), b) = R.0
    polynomial_eval(quadratic_polynomial(R.1, -(a + b), a * b), b) = R.0
}

/// The monic quadratic with roots `a` and `b` factorizes: its value at `x`
/// is the value of `X - b` times `x - a`.
theorem quadratic_eval_factorized[R: CommRing](a: R, b: R, x: R) {
    polynomial_eval(quadratic_polynomial(R.1, -(a + b), a * b), x) =
        polynomial_eval(linear_polynomial(R.1, -b), x) * (x - a)
} by {
    polynomial_linear_mul_quadratic(a, b)
    polynomial_mul(linear_polynomial(R.1, -a), linear_polynomial(R.1, -b)) =
        quadratic_polynomial(R.1, -(a + b), a * b)
    polynomial_eval(quadratic_polynomial(R.1, -(a + b), a * b), x) =
        polynomial_eval(polynomial_mul(linear_polynomial(R.1, -a), linear_polynomial(R.1, -b)), x)
    polynomial_eval_mul(linear_polynomial(R.1, -a), linear_polynomial(R.1, -b), x)
    polynomial_eval(polynomial_mul(linear_polynomial(R.1, -a), linear_polynomial(R.1, -b)), x) =
        polynomial_eval(linear_polynomial(R.1, -a), x) * polynomial_eval(linear_polynomial(R.1, -b), x)
    linear_polynomial_eval(R.1, -a, x)
    polynomial_eval(linear_polynomial(R.1, -a), x) = -a + R.1 * x
    R.1 * x = x
    polynomial_eval(linear_polynomial(R.1, -a), x) = -a + x
    -a + x = x + -a
    x + -a = x - a
    polynomial_eval(linear_polynomial(R.1, -a), x) = x - a
    polynomial_eval(quadratic_polynomial(R.1, -(a + b), a * b), x) =
        (x - a) * polynomial_eval(linear_polynomial(R.1, -b), x)
    (x - a) * polynomial_eval(linear_polynomial(R.1, -b), x) =
        polynomial_eval(linear_polynomial(R.1, -b), x) * (x - a)
    polynomial_eval(quadratic_polynomial(R.1, -(a + b), a * b), x) =
        polynomial_eval(linear_polynomial(R.1, -b), x) * (x - a)
}

/// The linear polynomial `X - b` witnesses the divisibility of the monic
/// quadratic with roots `a` and `b` by `X - a`.
theorem polynomial_linear_vieta_witness[R: CommRing](a: R, b: R) {
    polynomial_x_minus_a_polynomial_witness(
        quadratic_polynomial(R.1, -(a + b), a * b), a, linear_polynomial(R.1, -b))
} by {
    forall(x: R) {
        quadratic_eval_factorized(a, b, x)
        polynomial_eval(quadratic_polynomial(R.1, -(a + b), a * b), x) =
            polynomial_eval(linear_polynomial(R.1, -b), x) * (x - a)
        polynomial_x_minus_a_polynomial_witness_at(
            quadratic_polynomial(R.1, -(a + b), a * b), a, linear_polynomial(R.1, -b), x) =
            (polynomial_eval(quadratic_polynomial(R.1, -(a + b), a * b), x) =
                polynomial_eval(linear_polynomial(R.1, -b), x) * (x - a))
        polynomial_x_minus_a_polynomial_witness_at(
            quadratic_polynomial(R.1, -(a + b), a * b), a, linear_polynomial(R.1, -b), x)
    }
    polynomial_x_minus_a_polynomial_witness(
        quadratic_polynomial(R.1, -(a + b), a * b), a, linear_polynomial(R.1, -b)) =
        forall(x: R) {
            polynomial_x_minus_a_polynomial_witness_at(
                quadratic_polynomial(R.1, -(a + b), a * b), a, linear_polynomial(R.1, -b), x)
        }
    polynomial_x_minus_a_polynomial_witness(
        quadratic_polynomial(R.1, -(a + b), a * b), a, linear_polynomial(R.1, -b))
}

/// `X - a` divides the monic quadratic with roots `a` and `b`.
theorem polynomial_linear_vieta_divides[R: CommRing](a: R, b: R) {
    polynomial_divisible_by_x_minus_a_polynomial(
        quadratic_polynomial(R.1, -(a + b), a * b), a)
} by {
    polynomial_linear_vieta_witness(a, b)
    exists(q: Polynomial[R]) {
        q = linear_polynomial(R.1, -b) and polynomial_x_minus_a_polynomial_witness(
            quadratic_polynomial(R.1, -(a + b), a * b), a, q)
    }
    polynomial_divisible_by_x_minus_a_polynomial(
        quadratic_polynomial(R.1, -(a + b), a * b), a)
}

/// Shifting the evaluation point of a linear polynomial changes the constant
/// term: `p(x + t)` is linear with constant `b + a * t`.
theorem polynomial_linear_shift_eval[R: CommRing](a: R, b: R, t: R, x: R) {
    polynomial_eval(linear_polynomial(a, b + a * t), x) =
        polynomial_eval(linear_polynomial(a, b), x + t)
} by {
    linear_polynomial_eval(a, b + a * t, x)
    polynomial_eval(linear_polynomial(a, b + a * t), x) = (b + a * t) + a * x
    linear_polynomial_eval(a, b, x + t)
    polynomial_eval(linear_polynomial(a, b), x + t) = b + a * (x + t)
    a * (x + t) = a * x + a * t
    b + a * (x + t) = b + (a * x + a * t)
    b + (a * x + a * t) = (b + a * t) + a * x
    (b + a * t) + a * x = b + a * (x + t)
    polynomial_eval(linear_polynomial(a, b + a * t), x) =
        polynomial_eval(linear_polynomial(a, b), x + t)
}

/// Composing linear polynomials gives a linear polynomial: the value of
/// `a * (c * x + d) + b` is `(a * c) * x + (a * d + b)`.
theorem polynomial_linear_compose_eval[R: CommRing](a: R, b: R, c: R, d: R, x: R) {
    polynomial_eval(linear_polynomial(a, b), polynomial_eval(linear_polynomial(c, d), x)) =
        (b + a * d) + (a * c) * x
} by {
    linear_polynomial_eval(c, d, x)
    polynomial_eval(linear_polynomial(c, d), x) = d + c * x
    linear_polynomial_eval(a, b, polynomial_eval(linear_polynomial(c, d), x))
    polynomial_eval(linear_polynomial(a, b), polynomial_eval(linear_polynomial(c, d), x)) =
        b + a * polynomial_eval(linear_polynomial(c, d), x)
    b + a * polynomial_eval(linear_polynomial(c, d), x) = b + a * (d + c * x)
    a * (d + c * x) = a * d + a * (c * x)
    b + a * (d + c * x) = b + (a * d + a * (c * x))
    b + (a * d + a * (c * x)) = (b + a * d) + a * (c * x)
    a * (c * x) = (a * c) * x
    (b + a * d) + a * (c * x) = (b + a * d) + (a * c) * x
    polynomial_eval(linear_polynomial(a, b), polynomial_eval(linear_polynomial(c, d), x)) =
        (b + a * d) + (a * c) * x
}

/// The constant coefficient of a quadratic polynomial.
theorem quadratic_polynomial_coeff_zero[R: Semiring](a: R, b: R, c: R) {
    quadratic_polynomial(a, b, c).coeff(Nat.0) = c
} by {
    polynomial_add_coeff(polynomial_constant(c) + polynomial_monomial(Nat.1, b),
        polynomial_monomial(Nat.2, a), Nat.0)
    quadratic_polynomial(a, b, c).coeff(Nat.0) =
        (polynomial_constant(c) + polynomial_monomial(Nat.1, b)).coeff(Nat.0) +
        polynomial_monomial(Nat.2, a).coeff(Nat.0)
    polynomial_add_coeff(polynomial_constant(c), polynomial_monomial(Nat.1, b), Nat.0)
    (polynomial_constant(c) + polynomial_monomial(Nat.1, b)).coeff(Nat.0) =
        polynomial_constant(c).coeff(Nat.0) + polynomial_monomial(Nat.1, b).coeff(Nat.0)
    polynomial_constant_coeff_zero(c)
    polynomial_constant(c).coeff(Nat.0) = c
    Nat.0 != Nat.1
    polynomial_monomial_coeff_of_ne(Nat.1, b, Nat.0)
    Polynomial[R].monomial(Nat.1, b) = polynomial_monomial(Nat.1, b)
    polynomial_monomial(Nat.1, b).coeff(Nat.0) = R.0
    Nat.0 != Nat.2
    polynomial_monomial_coeff_of_ne(Nat.2, a, Nat.0)
    Polynomial[R].monomial(Nat.2, a) = polynomial_monomial(Nat.2, a)
    polynomial_monomial(Nat.2, a).coeff(Nat.0) = R.0
    quadratic_polynomial(a, b, c).coeff(Nat.0) = c + R.0 + R.0
    c + R.0 + R.0 = c
}

/// The linear coefficient of a quadratic polynomial.
theorem quadratic_polynomial_coeff_one[R: Semiring](a: R, b: R, c: R) {
    quadratic_polynomial(a, b, c).coeff(Nat.1) = b
} by {
    polynomial_add_coeff(polynomial_constant(c) + polynomial_monomial(Nat.1, b),
        polynomial_monomial(Nat.2, a), Nat.1)
    quadratic_polynomial(a, b, c).coeff(Nat.1) =
        (polynomial_constant(c) + polynomial_monomial(Nat.1, b)).coeff(Nat.1) +
        polynomial_monomial(Nat.2, a).coeff(Nat.1)
    polynomial_add_coeff(polynomial_constant(c), polynomial_monomial(Nat.1, b), Nat.1)
    (polynomial_constant(c) + polynomial_monomial(Nat.1, b)).coeff(Nat.1) =
        polynomial_constant(c).coeff(Nat.1) + polynomial_monomial(Nat.1, b).coeff(Nat.1)
    Nat.1 != Nat.0
    polynomial_constant_coeff_of_ne_zero(c, Nat.1)
    polynomial_constant(c).coeff(Nat.1) = R.0
    polynomial_monomial_coeff_self(Nat.1, b)
    Polynomial[R].monomial(Nat.1, b) = polynomial_monomial(Nat.1, b)
    polynomial_monomial(Nat.1, b).coeff(Nat.1) = b
    Nat.1 != Nat.2
    polynomial_monomial_coeff_of_ne(Nat.2, a, Nat.1)
    Polynomial[R].monomial(Nat.2, a) = polynomial_monomial(Nat.2, a)
    polynomial_monomial(Nat.2, a).coeff(Nat.1) = R.0
    quadratic_polynomial(a, b, c).coeff(Nat.1) = R.0 + b + R.0
    R.0 + b + R.0 = b
}

/// The quadratic coefficient of a quadratic polynomial.
theorem quadratic_polynomial_coeff_two[R: Semiring](a: R, b: R, c: R) {
    quadratic_polynomial(a, b, c).coeff(Nat.2) = a
} by {
    polynomial_add_coeff(polynomial_constant(c) + polynomial_monomial(Nat.1, b),
        polynomial_monomial(Nat.2, a), Nat.2)
    quadratic_polynomial(a, b, c).coeff(Nat.2) =
        (polynomial_constant(c) + polynomial_monomial(Nat.1, b)).coeff(Nat.2) +
        polynomial_monomial(Nat.2, a).coeff(Nat.2)
    polynomial_add_coeff(polynomial_constant(c), polynomial_monomial(Nat.1, b), Nat.2)
    (polynomial_constant(c) + polynomial_monomial(Nat.1, b)).coeff(Nat.2) =
        polynomial_constant(c).coeff(Nat.2) + polynomial_monomial(Nat.1, b).coeff(Nat.2)
    Nat.2 != Nat.0
    polynomial_constant_coeff_of_ne_zero(c, Nat.2)
    polynomial_constant(c).coeff(Nat.2) = R.0
    Nat.1 != Nat.2
    polynomial_monomial_coeff_of_ne(Nat.1, b, Nat.2)
    Polynomial[R].monomial(Nat.1, b) = polynomial_monomial(Nat.1, b)
    polynomial_monomial(Nat.1, b).coeff(Nat.2) = R.0
    polynomial_monomial_coeff_self(Nat.2, a)
    Polynomial[R].monomial(Nat.2, a) = polynomial_monomial(Nat.2, a)
    polynomial_monomial(Nat.2, a).coeff(Nat.2) = a
    quadratic_polynomial(a, b, c).coeff(Nat.2) = R.0 + R.0 + a
    R.0 + R.0 + a = a
}

/// The simple Vieta relations: the monic quadratic with roots `r1` and `r2`
/// has linear coefficient `-(r1 + r2)`.
theorem vieta_quadratic_coeff_linear[R: CommRing](r1: R, r2: R) {
    quadratic_polynomial(R.1, -(r1 + r2), r1 * r2).coeff(Nat.1) = -(r1 + r2)
} by {
    quadratic_polynomial_coeff_one(R.1, -(r1 + r2), r1 * r2)
}

/// The simple Vieta relations: the monic quadratic with roots `r1` and `r2`
/// has constant coefficient `r1 * r2`.
theorem vieta_quadratic_coeff_constant[R: CommRing](r1: R, r2: R) {
    quadratic_polynomial(R.1, -(r1 + r2), r1 * r2).coeff(Nat.0) = r1 * r2
} by {
    quadratic_polynomial_coeff_zero(R.1, -(r1 + r2), r1 * r2)
}

/// The simple Vieta relations: the monic quadratic with roots `r1` and `r2`
/// has leading coefficient one.
theorem vieta_quadratic_coeff_leading[R: CommRing](r1: R, r2: R) {
    quadratic_polynomial(R.1, -(r1 + r2), r1 * r2).coeff(Nat.2) = R.1
} by {
    quadratic_polynomial_coeff_two(R.1, -(r1 + r2), r1 * r2)
}
