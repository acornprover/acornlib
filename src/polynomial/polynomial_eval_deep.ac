/// Deepening evaluation algebra for univariate polynomials: the value at
/// zero, values of monomials, and evaluation-level consequences of the
/// polynomial ring laws.

from nat import Nat, alt_induction, alt_suc_ne_zero, lt_suc, lt_or_lte, lt_and_lte, lt_not_symm,
    zero_or_suc, not_lt_zero
from semiring import Semiring
from comm_ring import CommRing
from algebra.semigroup import Semigroup
from algebra.comm_semigroup import CommSemigroup
from data.basic.functions import function_extensionality
from polynomial import Polynomial, polynomial_monomial, polynomial_constant,
    polynomial_monomial_coeff, polynomial_monomial_coeff_self, polynomial_monomial_coeff_of_ne,
    polynomial_zero_coeff, polynomial_ext_pointwise, polynomial_eq_zero_of_coeff_zero,
    coeff_eval, coeff_tail, coeff_zero_from, coeff_zero_from_at, coeff_zero_at,
    polynomial_support_bounded_by, polynomial_support_bound_exists,
    polynomial_support_bounded_by_apply, polynomial_eval_bound, polynomial_eval_bound_eq_coeff_eval,
    polynomial_eval, polynomial_eval_eq_eval_bound_of_support_bounded, polynomial_eval_zero,
    polynomial_eval_constant, polynomial_eval_add, polynomial_eval_mul, polynomial_mul
from polynomial_mul_comm import polynomial_mul_comm, polynomial_mul_one_right, polynomial_mul_one_left
from polynomial_mul_distrib import polynomial_mul_add_left
from polynomial.polynomial_ring_theory import polynomial_mul_add_right

numerals Nat

/// Equal polynomials evaluate equally at every point.
lemma polynomial_eval_eq_of_eq[R: Semiring](p: Polynomial[R], q: Polynomial[R], x: R) {
    p = q implies polynomial_eval(p, x) = polynomial_eval(q, x)
} by {
    if p = q {
        polynomial_eval(p, x) = polynomial_eval(q, x)
    }
}

/// A ring-arithmetic helper: `a * (b * c) = b * (a * c)` in a commutative
/// ring.
lemma eval_deep_mul_swap_assoc[R: CommRing](a: R, b: R, c: R) {
    a * (b * c) = b * (a * c)
} by {
    Semigroup.mul_associative[R](a, b, c)
    a * (b * c) = (a * b) * c
    CommSemigroup.commutative[R](a, b)
    a * b = b * a
    (a * b) * c = (b * a) * c
    Semigroup.mul_associative[R](b, a, c)
    b * (a * c) = (b * a) * c
    a * (b * c) = b * (a * c)
}

/// Horner evaluation at zero with at least one coefficient slot returns the
/// constant coefficient.
lemma coeff_eval_zero_point_suc[R: Semiring](c: Nat -> R, n: Nat) {
    coeff_eval(c, R.0, n.suc) = c(Nat.0)
} by {
    coeff_eval(c, R.0, n.suc) = c(Nat.0) + R.0 * coeff_eval(coeff_tail(c), R.0, n)
    R.0 * coeff_eval(coeff_tail(c), R.0, n) = R.0
    c(Nat.0) + R.0 * coeff_eval(coeff_tail(c), R.0, n) = c(Nat.0) + R.0
    c(Nat.0) + R.0 = c(Nat.0)
    coeff_eval(c, R.0, n.suc) = c(Nat.0)
}

/// Evaluating a polynomial at zero returns its constant coefficient.
theorem polynomial_eval_zero_point[R: Semiring](p: Polynomial[R]) {
    polynomial_eval(p, R.0) = p.coeff(Nat.0)
} by {
    polynomial_support_bound_exists(p)
    let n: Nat satisfy {
        polynomial_support_bounded_by(p, n)
    }
    if n = Nat.0 {
        forall(k: Nat) {
            not_lt_zero(k)
            polynomial_support_bounded_by_apply(p, n, k)
            p.coeff(k) = R.0
        }
        polynomial_eq_zero_of_coeff_zero(p)
        p = Polynomial[R].zero
        polynomial_eval_eq_of_eq(p, Polynomial[R].zero, R.0)
        polynomial_eval(p, R.0) = polynomial_eval(Polynomial[R].zero, R.0)
        polynomial_eval_zero(R.0)
        polynomial_eval(Polynomial[R].zero, R.0) = R.0
        polynomial_eval(p, R.0) = R.0
        not_lt_zero(Nat.0)
        polynomial_support_bounded_by_apply(p, n, Nat.0)
        p.coeff(Nat.0) = R.0
        polynomial_eval(p, R.0) = p.coeff(Nat.0)
    } else {
        zero_or_suc(n)
        let m: Nat satisfy {
            n = m.suc
        }
        polynomial_eval_eq_eval_bound_of_support_bounded(p, R.0, n)
        polynomial_eval(p, R.0) = polynomial_eval_bound(p, R.0, n)
        polynomial_eval_bound_eq_coeff_eval(p, R.0, n)
        polynomial_eval_bound(p, R.0, n) = coeff_eval(p.coeff, R.0, n)
        n = m.suc
        coeff_eval(p.coeff, R.0, n) = coeff_eval(p.coeff, R.0, m.suc)
        coeff_eval_zero_point_suc(p.coeff, m)
        coeff_eval(p.coeff, R.0, m.suc) = p.coeff(Nat.0)
        polynomial_eval(p, R.0) = p.coeff(Nat.0)
    }
}

/// A monomial is support-bounded by one more than its exponent.
theorem polynomial_monomial_support_bounded_by_suc[R: Semiring](n: Nat, r: R) {
    polynomial_support_bounded_by(Polynomial[R].monomial(n, r), n.suc)
} by {
    forall(k: Nat) {
        if k < n.suc {
            coeff_zero_from_at(Polynomial[R].monomial(n, r).coeff, n.suc, k) =
                (not k < n.suc implies coeff_zero_at(Polynomial[R].monomial(n, r).coeff, k))
            coeff_zero_from_at(Polynomial[R].monomial(n, r).coeff, n.suc, k)
        } else {
            not k < n.suc
            lt_or_lte(k, n.suc)
            n.suc <= k
            if k = n {
                lt_suc(n)
                n < n.suc
                lt_and_lte(n, n.suc, k)
                n < k
                lt_not_symm(n, k)
                not k < n
                lt_and_lte(k, n, k)
                k < k
                lt_not_symm(k, k)
                not k < k
                false
            }
            k != n
            polynomial_monomial_coeff_of_ne[R](n, r, k)
            Polynomial[R].monomial(n, r).coeff(k) = R.0
            coeff_zero_at(Polynomial[R].monomial(n, r).coeff, k) =
                (Polynomial[R].monomial(n, r).coeff(k) = R.0)
            coeff_zero_at(Polynomial[R].monomial(n, r).coeff, k)
            coeff_zero_from_at(Polynomial[R].monomial(n, r).coeff, n.suc, k) =
                (not k < n.suc implies coeff_zero_at(Polynomial[R].monomial(n, r).coeff, k))
            coeff_zero_from_at(Polynomial[R].monomial(n, r).coeff, n.suc, k)
        }
    }
    coeff_zero_from(Polynomial[R].monomial(n, r).coeff, n.suc) = forall(k: Nat) {
        coeff_zero_from_at(Polynomial[R].monomial(n, r).coeff, n.suc, k)
    }
    coeff_zero_from(Polynomial[R].monomial(n, r).coeff, n.suc)
    polynomial_support_bounded_by(Polynomial[R].monomial(n, r), n.suc) =
        coeff_zero_from(Polynomial[R].monomial(n, r).coeff, n.suc)
    polynomial_support_bounded_by(Polynomial[R].monomial(n, r), n.suc)
}

/// A helper: two functions agreeing at every index are equal.
lemma function_extensionality_nat[R: Semiring](f: Nat -> R, g: Nat -> R) {
    (forall(i: Nat) { f(i) = g(i) }) implies f = g
} by {
    if forall(i: Nat) { f(i) = g(i) } {
        function_extensionality(f, g)
        f = g
    }
}

/// The tail of a monomial coefficient function is the coefficient function of
/// the monomial with exponent one less.
lemma coeff_tail_monomial_coeff[R: Semiring](n: Nat, r: R) {
    coeff_tail(polynomial_monomial_coeff[R](n.suc, r)) = polynomial_monomial_coeff[R](n, r)
} by {
    forall(i: Nat) {
        coeff_tail(polynomial_monomial_coeff[R](n.suc, r), i) =
            polynomial_monomial_coeff[R](n.suc, r, i.suc)
        if i = n {
            i.suc = n.suc
            polynomial_monomial_coeff[R](n.suc, r, i.suc) = r
            polynomial_monomial_coeff[R](n, r, i) = r
            coeff_tail(polynomial_monomial_coeff[R](n.suc, r), i) =
                polynomial_monomial_coeff[R](n, r, i)
        } else {
            i != n
            if i.suc = n.suc {
                i = n
                false
            }
            i.suc != n.suc
            polynomial_monomial_coeff[R](n.suc, r, i.suc) = R.0
            polynomial_monomial_coeff[R](n, r, i) = R.0
            coeff_tail(polynomial_monomial_coeff[R](n.suc, r), i) =
                polynomial_monomial_coeff[R](n, r, i)
        }
    }
    function_extensionality_nat(coeff_tail(polynomial_monomial_coeff[R](n.suc, r)),
        polynomial_monomial_coeff[R](n, r))
}

/// Horner evaluation of a monomial coefficient function with one more slot
/// than the exponent is the monomial value.
lemma coeff_eval_monomial_coeff_suc[R: CommRing](n: Nat, r: R, x: R) {
    coeff_eval(polynomial_monomial_coeff[R](n, r), x, n.suc) = r * x.pow(n)
} by {
    define statement(k: Nat) -> Bool {
        coeff_eval(polynomial_monomial_coeff[R](k, r), x, k.suc) = r * x.pow(k)
    }

    coeff_eval(polynomial_monomial_coeff[R](Nat.0, r), x, Nat.0.suc) =
        polynomial_monomial_coeff[R](Nat.0, r, Nat.0) +
        x * coeff_eval(coeff_tail(polynomial_monomial_coeff[R](Nat.0, r)), x, Nat.0)
    polynomial_monomial_coeff[R](Nat.0, r, Nat.0) = r
    coeff_eval(coeff_tail(polynomial_monomial_coeff[R](Nat.0, r)), x, Nat.0) = R.0
    x * R.0 = R.0
    polynomial_monomial_coeff[R](Nat.0, r, Nat.0) +
        x * coeff_eval(coeff_tail(polynomial_monomial_coeff[R](Nat.0, r)), x, Nat.0) =
        r + R.0
    r + R.0 = r
    coeff_eval(polynomial_monomial_coeff[R](Nat.0, r), x, Nat.0.suc) = r
    x.pow(Nat.0) = R.1
    r * x.pow(Nat.0) = r * R.1
    r * R.1 = r
    coeff_eval(polynomial_monomial_coeff[R](Nat.0, r), x, Nat.0.suc) = r * x.pow(Nat.0)
    statement(Nat.0)

    forall(m: Nat) {
        if statement(m) {
            coeff_eval(polynomial_monomial_coeff[R](m.suc, r), x, m.suc.suc) =
                polynomial_monomial_coeff[R](m.suc, r, Nat.0) +
                x * coeff_eval(coeff_tail(polynomial_monomial_coeff[R](m.suc, r)), x, m.suc)
            alt_suc_ne_zero(m)
            m.suc != Nat.0
            Nat.0 != m.suc
            polynomial_monomial_coeff[R](m.suc, r, Nat.0) = R.0
            coeff_tail_monomial_coeff(m, r)
            coeff_eval(coeff_tail(polynomial_monomial_coeff[R](m.suc, r)), x, m.suc) =
                coeff_eval(polynomial_monomial_coeff[R](m, r), x, m.suc)
            statement(m) = (coeff_eval(polynomial_monomial_coeff[R](m, r), x, m.suc) =
                r * x.pow(m))
            coeff_eval(polynomial_monomial_coeff[R](m, r), x, m.suc) = r * x.pow(m)
            coeff_eval(polynomial_monomial_coeff[R](m.suc, r), x, m.suc.suc) =
                R.0 + x * (r * x.pow(m))
            R.0 + x * (r * x.pow(m)) = x * (r * x.pow(m))
            eval_deep_mul_swap_assoc(x, r, x.pow(m))
            x * (r * x.pow(m)) = r * (x * x.pow(m))
            x.pow(m.suc) = x * x.pow(m)
            r * (x * x.pow(m)) = r * x.pow(m.suc)
            coeff_eval(polynomial_monomial_coeff[R](m.suc, r), x, m.suc.suc) =
                r * x.pow(m.suc)
            statement(m.suc)
        }
    }

    statement(Nat.0) and forall(m: Nat) { statement(m) implies statement(m.suc) }
    alt_induction(statement)
    forall(m: Nat) { statement(m) }
    statement(n)
}

/// The value of a monomial `r * X^n` at a point is `r * x^n`.
theorem polynomial_eval_monomial[R: CommRing](n: Nat, r: R, x: R) {
    polynomial_eval(Polynomial[R].monomial(n, r), x) = r * x.pow(n)
} by {
    polynomial_monomial_support_bounded_by_suc(n, r)
    polynomial_support_bounded_by(Polynomial[R].monomial(n, r), n.suc)
    polynomial_eval_eq_eval_bound_of_support_bounded(Polynomial[R].monomial(n, r), x, n.suc)
    polynomial_eval(Polynomial[R].monomial(n, r), x) =
        polynomial_eval_bound(Polynomial[R].monomial(n, r), x, n.suc)
    polynomial_eval_bound_eq_coeff_eval(Polynomial[R].monomial(n, r), x, n.suc)
    polynomial_eval_bound(Polynomial[R].monomial(n, r), x, n.suc) =
        coeff_eval(Polynomial[R].monomial(n, r).coeff, x, n.suc)
    Polynomial[R].new(polynomial_monomial_coeff[R](n, r)) =
        Option.some(Polynomial[R].monomial(n, r))
    Polynomial[R].monomial(n, r).coeff = polynomial_monomial_coeff[R](n, r)
    coeff_eval(Polynomial[R].monomial(n, r).coeff, x, n.suc) =
        coeff_eval(polynomial_monomial_coeff[R](n, r), x, n.suc)
    coeff_eval_monomial_coeff_suc(n, r, x)
    coeff_eval(polynomial_monomial_coeff[R](n, r), x, n.suc) = r * x.pow(n)
    polynomial_eval(Polynomial[R].monomial(n, r), x) = r * x.pow(n)
}

/// A constant polynomial vanishes at a point exactly when its coefficient is
/// zero.
theorem polynomial_eval_constant_eq_zero[R: Semiring](r: R, x: R) {
    (polynomial_eval(Polynomial[R].constant(r), x) = R.0) = (r = R.0)
} by {
    if polynomial_eval(Polynomial[R].constant(r), x) = R.0 {
        polynomial_eval_constant(r, x)
        polynomial_eval(Polynomial[R].constant(r), x) = r
        r = R.0
    }
    if r = R.0 {
        polynomial_eval(Polynomial[R].constant(r), x) =
            polynomial_eval(Polynomial[R].constant(R.0), x)
        polynomial_eval_constant(R.0, x)
        polynomial_eval(Polynomial[R].constant(R.0), x) = R.0
        polynomial_eval(Polynomial[R].constant(r), x) = R.0
    }
}

/// A scalar multiple evaluates as the scalar times the value.
theorem polynomial_eval_constant_mul[R: CommRing](r: R, p: Polynomial[R], x: R) {
    polynomial_eval(polynomial_mul(polynomial_constant(r), p), x) =
        r * polynomial_eval(p, x)
} by {
    polynomial_eval_mul(polynomial_constant(r), p, x)
    polynomial_eval(polynomial_mul(polynomial_constant(r), p), x) =
        polynomial_eval(polynomial_constant(r), x) * polynomial_eval(p, x)
    polynomial_eval_constant(r, x)
    polynomial_eval(polynomial_constant(r), x) = r
    polynomial_eval(polynomial_mul(polynomial_constant(r), p), x) =
        r * polynomial_eval(p, x)
}

/// Evaluation distributes multiplication over addition of the second factor.
theorem polynomial_eval_mul_add_distributive[R: CommRing](
    p: Polynomial[R], q: Polynomial[R], r: Polynomial[R], x: R
) {
    polynomial_eval(polynomial_mul(p, q + r), x) =
        polynomial_eval(p, x) * (polynomial_eval(q, x) + polynomial_eval(r, x))
} by {
    polynomial_mul_add_left(p, q, r)
    polynomial_mul(p, q + r) = polynomial_mul(p, q) + polynomial_mul(p, r)
    polynomial_eval_eq_of_eq(polynomial_mul(p, q + r),
        polynomial_mul(p, q) + polynomial_mul(p, r), x)
    polynomial_eval(polynomial_mul(p, q + r), x) =
        polynomial_eval(polynomial_mul(p, q) + polynomial_mul(p, r), x)
    polynomial_eval_add(polynomial_mul(p, q), polynomial_mul(p, r), x)
    polynomial_eval(polynomial_mul(p, q) + polynomial_mul(p, r), x) =
        polynomial_eval(polynomial_mul(p, q), x) + polynomial_eval(polynomial_mul(p, r), x)
    polynomial_eval_mul(p, q, x)
    polynomial_eval(polynomial_mul(p, q), x) = polynomial_eval(p, x) * polynomial_eval(q, x)
    polynomial_eval_mul(p, r, x)
    polynomial_eval(polynomial_mul(p, r), x) = polynomial_eval(p, x) * polynomial_eval(r, x)
    polynomial_eval(p, x) * polynomial_eval(q, x) + polynomial_eval(p, x) * polynomial_eval(r, x) =
        polynomial_eval(p, x) * (polynomial_eval(q, x) + polynomial_eval(r, x))
    polynomial_eval(polynomial_mul(p, q + r), x) =
        polynomial_eval(p, x) * (polynomial_eval(q, x) + polynomial_eval(r, x))
}

/// Evaluation distributes multiplication over addition of the first factor.
theorem polynomial_eval_add_mul_distributive[R: CommRing](
    p: Polynomial[R], q: Polynomial[R], r: Polynomial[R], x: R
) {
    polynomial_eval(polynomial_mul(p + q, r), x) =
        (polynomial_eval(p, x) + polynomial_eval(q, x)) * polynomial_eval(r, x)
} by {
    polynomial_mul_add_right(p, q, r)
    polynomial_mul(p + q, r) = polynomial_mul(p, r) + polynomial_mul(q, r)
    polynomial_eval_eq_of_eq(polynomial_mul(p + q, r),
        polynomial_mul(p, r) + polynomial_mul(q, r), x)
    polynomial_eval(polynomial_mul(p + q, r), x) =
        polynomial_eval(polynomial_mul(p, r) + polynomial_mul(q, r), x)
    polynomial_eval_add(polynomial_mul(p, r), polynomial_mul(q, r), x)
    polynomial_eval(polynomial_mul(p, r) + polynomial_mul(q, r), x) =
        polynomial_eval(polynomial_mul(p, r), x) + polynomial_eval(polynomial_mul(q, r), x)
    polynomial_eval_mul(p, r, x)
    polynomial_eval(polynomial_mul(p, r), x) = polynomial_eval(p, x) * polynomial_eval(r, x)
    polynomial_eval_mul(q, r, x)
    polynomial_eval(polynomial_mul(q, r), x) = polynomial_eval(q, x) * polynomial_eval(r, x)
    polynomial_eval(p, x) * polynomial_eval(r, x) + polynomial_eval(q, x) * polynomial_eval(r, x) =
        (polynomial_eval(p, x) + polynomial_eval(q, x)) * polynomial_eval(r, x)
    polynomial_eval(polynomial_mul(p + q, r), x) =
        (polynomial_eval(p, x) + polynomial_eval(q, x)) * polynomial_eval(r, x)
}

/// The one polynomial is a right identity at the evaluation level.
theorem polynomial_eval_mul_one[R: CommRing](p: Polynomial[R], x: R) {
    polynomial_eval(polynomial_mul(p, Polynomial[R].one), x) = polynomial_eval(p, x)
} by {
    polynomial_mul_one_right(p)
    polynomial_mul(p, Polynomial[R].one) = p
    polynomial_eval_eq_of_eq(polynomial_mul(p, Polynomial[R].one), p, x)
    polynomial_eval(polynomial_mul(p, Polynomial[R].one), x) = polynomial_eval(p, x)
}

/// The one polynomial is a left identity at the evaluation level.
theorem polynomial_eval_one_mul[R: CommRing](p: Polynomial[R], x: R) {
    polynomial_eval(polynomial_mul(Polynomial[R].one, p), x) = polynomial_eval(p, x)
} by {
    polynomial_mul_one_left(p)
    polynomial_mul(Polynomial[R].one, p) = p
    polynomial_eval_eq_of_eq(polynomial_mul(Polynomial[R].one, p), p, x)
    polynomial_eval(polynomial_mul(Polynomial[R].one, p), x) = polynomial_eval(p, x)
}

/// The value of a linear combination of polynomials is the corresponding
/// linear combination of values.
theorem polynomial_eval_linear_combination[R: CommRing](
    r1: R, r2: R, p: Polynomial[R], q: Polynomial[R], x: R
) {
    polynomial_eval(polynomial_mul(polynomial_constant(r1), p) +
        polynomial_mul(polynomial_constant(r2), q), x) =
        r1 * polynomial_eval(p, x) + r2 * polynomial_eval(q, x)
} by {
    polynomial_eval_add(polynomial_mul(polynomial_constant(r1), p),
        polynomial_mul(polynomial_constant(r2), q), x)
    polynomial_eval(polynomial_mul(polynomial_constant(r1), p) +
        polynomial_mul(polynomial_constant(r2), q), x) =
        polynomial_eval(polynomial_mul(polynomial_constant(r1), p), x) +
        polynomial_eval(polynomial_mul(polynomial_constant(r2), q), x)
    polynomial_eval_constant_mul(r1, p, x)
    polynomial_eval(polynomial_mul(polynomial_constant(r1), p), x) =
        r1 * polynomial_eval(p, x)
    polynomial_eval_constant_mul(r2, q, x)
    polynomial_eval(polynomial_mul(polynomial_constant(r2), q), x) =
        r2 * polynomial_eval(q, x)
    polynomial_eval(polynomial_mul(polynomial_constant(r1), p) +
        polynomial_mul(polynomial_constant(r2), q), x) =
        r1 * polynomial_eval(p, x) + r2 * polynomial_eval(q, x)
}

/// Scalar multiplication commutes at the evaluation level: `r * p` and `p * r`
/// have the same value.
theorem polynomial_eval_scalar_mul_comm[R: CommRing](r: R, p: Polynomial[R], x: R) {
    polynomial_eval(polynomial_mul(polynomial_constant(r), p), x) =
        polynomial_eval(polynomial_mul(p, polynomial_constant(r)), x)
} by {
    polynomial_mul_comm(polynomial_constant(r), p)
    polynomial_mul(polynomial_constant(r), p) = polynomial_mul(p, polynomial_constant(r))
    polynomial_eval_eq_of_eq(polynomial_mul(polynomial_constant(r), p),
        polynomial_mul(p, polynomial_constant(r)), x)
    polynomial_eval(polynomial_mul(polynomial_constant(r), p), x) =
        polynomial_eval(polynomial_mul(p, polynomial_constant(r)), x)
}
