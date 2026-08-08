/// Inductive infrastructure for bounding polynomial roots over fields.

from algebra.add_group import right_cancel
from algebra.field.field import Field, field_mul_eq_zero
from list import List, unique_implies_tail_unique, unique_length
from nat import Nat, alt_induction, lt_cancel_suc, lt_imp_lte_suc, lt_suc,
    lt_suc_right, lte_and_lt, trichotomy, zero_or_suc
from polynomial.add_monoid_algebra_bridge import coeff_zero_at, coeff_zero_from, coeff_zero_from_at
from polynomial.base import Polynomial, polynomial_coeff_zero_of_eq_zero,
    polynomial_eq_zero_of_coeff_zero
from polynomial.coeff_eval import coeff_eval, coeff_quotient, coeff_quotient_zero_index, coeff_tail
from polynomial.eval import polynomial_quotient, polynomial_quotient_coeff_at,
    polynomial_eval_bound, polynomial_support_bounded_by, polynomial_support_bounded_by_apply,
    polynomial_quotient_support_bounded_by
from polynomial.global_eval import polynomial_eval, polynomial_eval_eq_eval_bound_of_support_bounded
from polynomial.global_factor import polynomial_quotient_global_witness_of_support_bounded,
    polynomial_x_minus_a_polynomial_witness_apply

lemma root_bound_coeff_zero_from_at_intro[F: Field](c: Nat -> F, n: Nat, k: Nat) {
    not k < n and c(k) = F.0 implies coeff_zero_from_at(c, n, k)
} by {
    if not k < n and c(k) = F.0 {
        coeff_zero_at(c, k)
        coeff_zero_from_at(c, n, k) = (not k < n implies coeff_zero_at(c, k))
        coeff_zero_from_at(c, n, k)
    }
}

lemma root_bound_coeff_zero_from_intro[F: Field](c: Nat -> F, n: Nat) {
    (forall(k: Nat) { coeff_zero_from_at(c, n, k) }) implies coeff_zero_from(c, n)
} by {
    forall(k: Nat) {
        coeff_zero_from_at(c, n, k)
    }
    coeff_zero_from(c, n) = forall(k: Nat) {
        coeff_zero_from_at(c, n, k)
    }
}

/// The last possible coefficient of a synthetic quotient is zero.
lemma coeff_quotient_diagonal_zero[R: Field](c: Nat -> R, a: R, n: Nat) {
    coeff_quotient(c, a, n.suc, n) = R.0
} by {
    define statement(k: Nat) -> Bool {
        forall(d: Nat -> R, b: R) {
            coeff_quotient(d, b, k.suc, k) = R.0
        }
    }

    forall(d: Nat -> R, b: R) {
        coeff_quotient(d, b, Nat.0.suc, Nat.0) =
            coeff_eval(coeff_tail(d), b, Nat.0)
        coeff_eval(coeff_tail(d), b, Nat.0) = R.0
        coeff_quotient(d, b, Nat.0.suc, Nat.0) = R.0
    }
    statement(Nat.0)

    forall(k: Nat) {
        if statement(k) {
            forall(d: Nat -> R, b: R) {
                coeff_quotient(d, b, k.suc.suc, k.suc) =
                    coeff_quotient(coeff_tail(d), b, k.suc, k)
                statement(k) = forall(e: Nat -> R, x: R) {
                    coeff_quotient(e, x, k.suc, k) = R.0
                }
                coeff_quotient(coeff_tail(d), b, k.suc, k) = R.0
                coeff_quotient(d, b, k.suc.suc, k.suc) = R.0
            }
            statement(k.suc)
        }
    }
    statement(Nat.0) and forall(k: Nat) {
        statement(k) implies statement(k.suc)
    }
    alt_induction(statement)
    forall(k: Nat) { statement(k) }
    statement(n)
    statement(n) = forall(d: Nat -> R, b: R) {
        coeff_quotient(d, b, n.suc, n) = R.0
    }
    coeff_quotient(c, a, n.suc, n) = R.0
}

/// Adjacent synthetic quotient coefficients recover the intervening source coefficient.
lemma coeff_quotient_recurrence[F: Field](c: Nat -> F, a: F, n: Nat, i: Nat) {
    i < n implies
        coeff_quotient(c, a, n.suc, i) =
            c(i.suc) + a * coeff_quotient(c, a, n.suc, i.suc)
} by {
    define statement(k: Nat) -> Bool {
        forall(d: Nat -> F, b: F, j: Nat) {
            j < k implies
                coeff_quotient(d, b, k.suc, j) =
                    d(j.suc) + b * coeff_quotient(d, b, k.suc, j.suc)
        }
    }

    forall(d: Nat -> F, b: F, j: Nat) {
        if j < Nat.0 {
            false
        }
    }
    statement(Nat.0)

    forall(k: Nat) {
        if statement(k) {
            forall(d: Nat -> F, b: F, j: Nat) {
                if j < k.suc {
                    if j = Nat.0 {
                        coeff_quotient(d, b, k.suc.suc, Nat.0) =
                            coeff_eval(coeff_tail(d), b, k.suc)
                        coeff_eval(coeff_tail(d), b, k.suc) =
                            coeff_tail(d, Nat.0) + b *
                                coeff_eval(coeff_tail(coeff_tail(d)), b, k)
                        coeff_tail(d, Nat.0) = d(Nat.0.suc)
                        coeff_quotient(d, b, k.suc.suc, Nat.0.suc) =
                            coeff_quotient(coeff_tail(d), b, k.suc, Nat.0)
                        coeff_quotient(coeff_tail(d), b, k.suc, Nat.0) =
                            coeff_eval(coeff_tail(coeff_tail(d)), b, k)
                        coeff_quotient(d, b, k.suc.suc, j) =
                            d(j.suc) + b * coeff_quotient(d, b, k.suc.suc, j.suc)
                    } else {
                        zero_or_suc(j)
                        let jp: Nat satisfy {
                            j = jp.suc
                        }
                        j = jp.suc
                        j < k.suc
                        lt_cancel_suc(jp, k)
                        jp < k
                        statement(k) = forall(e: Nat -> F, x: F, m: Nat) {
                            m < k implies
                                coeff_quotient(e, x, k.suc, m) =
                                    e(m.suc) + x * coeff_quotient(e, x, k.suc, m.suc)
                        }
                        coeff_quotient(coeff_tail(d), b, k.suc, jp) =
                            coeff_tail(d, jp.suc) + b *
                                coeff_quotient(coeff_tail(d), b, k.suc, jp.suc)
                        coeff_quotient(d, b, k.suc.suc, jp.suc) =
                            coeff_quotient(coeff_tail(d), b, k.suc, jp)
                        coeff_quotient(d, b, k.suc.suc, jp.suc.suc) =
                            coeff_quotient(coeff_tail(d), b, k.suc, jp.suc)
                        coeff_tail(d, jp.suc) = d(jp.suc.suc)
                        coeff_quotient(d, b, k.suc.suc, j) =
                            d(j.suc) + b * coeff_quotient(d, b, k.suc.suc, j.suc)
                    }
                }
            }
            statement(k.suc)
        }
    }
    statement(Nat.0) and forall(k: Nat) {
        statement(k) implies statement(k.suc)
    }
    alt_induction(statement)
    forall(k: Nat) { statement(k) }
    statement(n)
    statement(n) = forall(d: Nat -> F, b: F, j: Nat) {
        j < n implies
            coeff_quotient(d, b, n.suc, j) =
                d(j.suc) + b * coeff_quotient(d, b, n.suc, j.suc)
    }
}

/// A quotient formed at a successor bound is supported below the predecessor.
theorem polynomial_quotient_support_bounded_by_pred[F: Field](
    p: Polynomial[F],
    a: F,
    n: Nat
) {
    polynomial_support_bounded_by(polynomial_quotient(p, a, n.suc), n)
} by {
    let q = polynomial_quotient(p, a, n.suc)
    polynomial_quotient_support_bounded_by(p, a, n.suc)
    polynomial_support_bounded_by(q, n.suc)
    forall(k: Nat) {
        if not k < n {
            trichotomy(n, k)
            if n < k {
                if k < n.suc {
                    lt_suc_right(k, n)
                    k = n or k < n
                    false
                }
                not k < n.suc
                polynomial_support_bounded_by_apply(q, n.suc, k)
                q.coeff(k) = F.0
            } else {
                not k < n
                not n < k
                n = k
                polynomial_quotient_coeff_at(p, a, n.suc, n)
                q.coeff(n) = coeff_quotient(p.coeff, a, n.suc, n)
                coeff_quotient_diagonal_zero(p.coeff, a, n)
                q.coeff(k) = F.0
            }
            root_bound_coeff_zero_from_at_intro(q.coeff, n, k)
            coeff_zero_from_at(q.coeff, n, k)
        }
    }
    root_bound_coeff_zero_from_intro(q.coeff, n)
    coeff_zero_from(q.coeff, n)
    polynomial_support_bounded_by(q, n) = coeff_zero_from(q.coeff, n)
    polynomial_support_bounded_by(q, n)
}

/// A rooted polynomial is zero if its synthetic quotient is zero at a valid
/// successor support bound.
theorem polynomial_eq_zero_of_root_quotient_eq_zero[F: Field](
    p: Polynomial[F],
    a: F,
    n: Nat
) {
    polynomial_support_bounded_by(p, n.suc) and
        polynomial_eval(p, a) = F.0 and
        polynomial_quotient(p, a, n.suc) = Polynomial[F].zero
    implies p = Polynomial[F].zero
} by {
    if polynomial_support_bounded_by(p, n.suc) and
        polynomial_eval(p, a) = F.0 and
        polynomial_quotient(p, a, n.suc) = Polynomial[F].zero {
        let q = polynomial_quotient(p, a, n.suc)
        q = Polynomial[F].zero
        polynomial_eval_eq_eval_bound_of_support_bounded(p, a, n.suc)
        polynomial_eval(p, a) = polynomial_eval_bound(p, a, n.suc)
        polynomial_eval_bound(p, a, n.suc) = F.0

        polynomial_coeff_zero_of_eq_zero(q, Nat.0)
        q.coeff(Nat.0) = F.0
        polynomial_quotient_coeff_at(p, a, n.suc, Nat.0)
        q.coeff(Nat.0) = coeff_quotient(p.coeff, a, n.suc, Nat.0)
        coeff_quotient_zero_index(p.coeff, a, n)
        coeff_quotient(p.coeff, a, n.suc, Nat.0) =
            coeff_eval(coeff_tail(p.coeff), a, n)
        coeff_eval(coeff_tail(p.coeff), a, n) = F.0
        polynomial_eval_bound(p, a, n.suc) = coeff_eval(p.coeff, a, n.suc)
        coeff_eval(p.coeff, a, n.suc) =
            p.coeff(Nat.0) + a * coeff_eval(coeff_tail(p.coeff), a, n)
        a * coeff_eval(coeff_tail(p.coeff), a, n) = F.0
        p.coeff(Nat.0) + a * coeff_eval(coeff_tail(p.coeff), a, n) =
            p.coeff(Nat.0) + F.0
        p.coeff(Nat.0) + F.0 = p.coeff(Nat.0)
        polynomial_eval_bound(p, a, n.suc) = p.coeff(Nat.0)
        p.coeff(Nat.0) = F.0

        forall(k: Nat) {
            if k = Nat.0 {
                p.coeff(k) = F.0
            } else {
                zero_or_suc(k)
                let kp: Nat satisfy {
                    k = kp.suc
                }
                k = kp.suc
                if kp < n {
                    coeff_quotient_recurrence(p.coeff, a, n, kp)
                    coeff_quotient(p.coeff, a, n.suc, kp) =
                        p.coeff(kp.suc) + a *
                            coeff_quotient(p.coeff, a, n.suc, kp.suc)
                    polynomial_coeff_zero_of_eq_zero(q, kp)
                    polynomial_coeff_zero_of_eq_zero(q, kp.suc)
                    q.coeff(kp) = F.0
                    q.coeff(kp.suc) = F.0
                    polynomial_quotient_coeff_at(p, a, n.suc, kp)
                    polynomial_quotient_coeff_at(p, a, n.suc, kp.suc)
                    q.coeff(kp) = coeff_quotient(p.coeff, a, n.suc, kp)
                    q.coeff(kp.suc) = coeff_quotient(p.coeff, a, n.suc, kp.suc)
                    coeff_quotient(p.coeff, a, n.suc, kp) = F.0
                    coeff_quotient(p.coeff, a, n.suc, kp.suc) = F.0
                    p.coeff(kp.suc) = F.0
                    p.coeff(k) = F.0
                } else {
                    not kp < n
                    if k < n.suc {
                        lt_cancel_suc(kp, n)
                        kp < n
                        false
                    }
                    not k < n.suc
                    polynomial_support_bounded_by_apply(p, n.suc, k)
                    p.coeff(k) = F.0
                }
            }
        }
        polynomial_eq_zero_of_coeff_zero(p)
        p = Polynomial[F].zero
    }
}

/// The synthetic quotient of a nonzero rooted polynomial remains nonzero.
theorem polynomial_quotient_ne_zero_of_root[F: Field](
    p: Polynomial[F],
    a: F,
    n: Nat
) {
    polynomial_support_bounded_by(p, n.suc) and
        p != Polynomial[F].zero and
        polynomial_eval(p, a) = F.0
    implies polynomial_quotient(p, a, n.suc) != Polynomial[F].zero
} by {
    if polynomial_support_bounded_by(p, n.suc) and
        p != Polynomial[F].zero and
        polynomial_eval(p, a) = F.0 {
        if polynomial_quotient(p, a, n.suc) = Polynomial[F].zero {
            polynomial_eq_zero_of_root_quotient_eq_zero(p, a, n)
            p = Polynomial[F].zero
            false
        }
    }
}

/// A list consists entirely of roots of a polynomial.
define polynomial_roots_on_list[F: Field](p: Polynomial[F], roots: List[F]) -> Bool {
    forall(x: F) {
        roots.contains(x) implies polynomial_eval(p, x) = F.0
    }
}

/// The root property for a list gives the root property for its tail.
theorem polynomial_roots_on_list_tail[F: Field](
    p: Polynomial[F],
    head: F,
    tail: List[F]
) {
    polynomial_roots_on_list(p, List.cons(head, tail)) implies
        polynomial_roots_on_list(p, tail)
} by {
    if polynomial_roots_on_list(p, List.cons(head, tail)) {
        polynomial_roots_on_list(p, List.cons(head, tail)) = forall(y: F) {
            List.cons(head, tail).contains(y) implies polynomial_eval(p, y) = F.0
        }
        forall(x: F) {
            if tail.contains(x) {
                List.cons(head, tail).contains(x)
                polynomial_eval(p, x) = F.0
            }
        }
        polynomial_roots_on_list(p, tail) = forall(x: F) {
            tail.contains(x) implies polynomial_eval(p, x) = F.0
        }
        polynomial_roots_on_list(p, tail)
    }
}

lemma polynomial_unique_cons_not_contains[F: Field](head: F, tail: List[F]) {
    List.cons(head, tail).is_unique implies not tail.contains(head)
} by {
    if List.cons(head, tail).is_unique {
        if tail.contains(head) {
            List.cons(head, tail).unique = List.cons(head, tail)
            tail.unique = List.cons(head, tail)
            unique_length(tail)
            tail.unique.length <= tail.length
            List.cons(head, tail).length = tail.length.suc
            tail.length.suc <= tail.length
            false
        }
    }
}

/// A root distinct from the selected root remains a root of the synthetic quotient.
theorem polynomial_quotient_root_of_distinct_root[F: Field](
    p: Polynomial[F],
    a: F,
    x: F,
    n: Nat
) {
    polynomial_support_bounded_by(p, n) and
        polynomial_eval(p, a) = F.0 and
        polynomial_eval(p, x) = F.0 and
        x != a
    implies polynomial_eval(polynomial_quotient(p, a, n), x) = F.0
} by {
    if polynomial_support_bounded_by(p, n) and
        polynomial_eval(p, a) = F.0 and
        polynomial_eval(p, x) = F.0 and
        x != a {
        let q = polynomial_quotient(p, a, n)
        polynomial_quotient_global_witness_of_support_bounded(p, a, n)
        polynomial_x_minus_a_polynomial_witness_apply(p, a, q, x)
        polynomial_eval(p, x) = polynomial_eval(q, x) * (x - a)
        polynomial_eval(q, x) * (x - a) = F.0
        field_mul_eq_zero(polynomial_eval(q, x), x - a)
        polynomial_eval(q, x) = F.0 or x - a = F.0
        if polynomial_eval(q, x) = F.0 {
            polynomial_eval(q, x) = F.0
        } else {
            x - a = F.0
            x + -a = F.0
            a + -a = F.0
            x + -a = a + -a
            right_cancel(-a, x, a)
            x = a
            false
        }
        polynomial_eval(q, x) = F.0
        q = polynomial_quotient(p, a, n)
        polynomial_eval(polynomial_quotient(p, a, n), x) = F.0
    }
}

/// After selecting the head root, every remaining distinct root is a root of
/// the synthetic quotient.
theorem polynomial_roots_on_list_quotient_tail[F: Field](
    p: Polynomial[F],
    head: F,
    tail: List[F],
    n: Nat
) {
    polynomial_support_bounded_by(p, n) and
        List.cons(head, tail).is_unique and
        polynomial_roots_on_list(p, List.cons(head, tail))
    implies polynomial_roots_on_list(polynomial_quotient(p, head, n), tail)
} by {
    if polynomial_support_bounded_by(p, n) and
        List.cons(head, tail).is_unique and
        polynomial_roots_on_list(p, List.cons(head, tail)) {
        polynomial_roots_on_list(p, List.cons(head, tail)) = forall(y: F) {
            List.cons(head, tail).contains(y) implies polynomial_eval(p, y) = F.0
        }
        List.cons(head, tail).contains(head)
        polynomial_eval(p, head) = F.0
        polynomial_unique_cons_not_contains(head, tail)
        not tail.contains(head)
        forall(x: F) {
            if tail.contains(x) {
                List.cons(head, tail).contains(x)
                polynomial_eval(p, x) = F.0
                if x = head {
                    tail.contains(head)
                    false
                }
                x != head
                polynomial_quotient_root_of_distinct_root(p, head, x, n)
                polynomial_eval(polynomial_quotient(p, head, n), x) = F.0
            }
        }
        polynomial_roots_on_list(polynomial_quotient(p, head, n), tail) = forall(x: F) {
            tail.contains(x) implies
                polynomial_eval(polynomial_quotient(p, head, n), x) = F.0
        }
        polynomial_roots_on_list(polynomial_quotient(p, head, n), tail)
    }
}

/// The assertion that every unique root list of a nonzero polynomial supported
/// below `n` has length strictly below `n`.
define polynomial_root_list_bound_property[F: Field](n: Nat) -> Bool {
    forall(p: Polynomial[F], roots: List[F]) {
        polynomial_support_bounded_by(p, n) and
            p != Polynomial[F].zero and
            roots.is_unique and
            polynomial_roots_on_list(p, roots)
        implies roots.length < n
    }
}

lemma polynomial_root_list_bound_property_all[F: Field](n: Nat) {
    polynomial_root_list_bound_property[F](n)
} by {
    define statement(k: Nat) -> Bool {
        polynomial_root_list_bound_property[F](k)
    }

    forall(p: Polynomial[F], roots: List[F]) {
        if polynomial_support_bounded_by(p, Nat.0) and
            p != Polynomial[F].zero and
            roots.is_unique and
            polynomial_roots_on_list(p, roots) {
            forall(i: Nat) {
                not i < Nat.0
                polynomial_support_bounded_by_apply(p, Nat.0, i)
                p.coeff(i) = F.0
            }
            polynomial_eq_zero_of_coeff_zero(p)
            p = Polynomial[F].zero
            false
        }
    }
    polynomial_root_list_bound_property[F](Nat.0)
    statement(Nat.0)

    forall(k: Nat) {
        if statement(k) {
            forall(p: Polynomial[F], roots: List[F]) {
                if polynomial_support_bounded_by(p, k.suc) and
                    p != Polynomial[F].zero and
                    roots.is_unique and
                    polynomial_roots_on_list(p, roots) {
                    match roots {
                        List.nil {
                            roots.length = Nat.0
                            lt_suc(k)
                            Nat.0 < k.suc
                            roots.length < k.suc
                        }
                        List.cons(head, tail) {
                            roots = List.cons(head, tail)
                            List.cons(head, tail).is_unique
                            polynomial_roots_on_list(p, List.cons(head, tail))
                            polynomial_roots_on_list(p, roots) = forall(x: F) {
                                roots.contains(x) implies polynomial_eval(p, x) = F.0
                            }
                            roots.contains(head)
                            polynomial_eval(p, head) = F.0
                            let q = polynomial_quotient(p, head, k.suc)
                            polynomial_quotient_support_bounded_by_pred(p, head, k)
                            polynomial_support_bounded_by(q, k)
                            polynomial_quotient_ne_zero_of_root(p, head, k)
                            q != Polynomial[F].zero
                            unique_implies_tail_unique(head, tail)
                            tail.is_unique
                            polynomial_roots_on_list_quotient_tail(p, head, tail, k.suc)
                            polynomial_roots_on_list(
                                polynomial_quotient(p, head, k.suc), tail)
                            q = polynomial_quotient(p, head, k.suc)
                            polynomial_roots_on_list(q, tail)
                            statement(k) = polynomial_root_list_bound_property[F](k)
                            polynomial_root_list_bound_property[F](k) =
                                forall(r: Polynomial[F], items: List[F]) {
                                    polynomial_support_bounded_by(r, k) and
                                        r != Polynomial[F].zero and
                                        items.is_unique and
                                        polynomial_roots_on_list(r, items)
                                    implies items.length < k
                                }
                            tail.length < k
                            lt_imp_lte_suc(tail.length, k)
                            tail.length.suc <= k
                            lt_suc(k)
                            k < k.suc
                            lte_and_lt(tail.length.suc, k, k.suc)
                            tail.length.suc < k.suc
                            roots.length = tail.length.suc
                            roots.length < k.suc
                        }
                    }
                }
            }
            polynomial_root_list_bound_property[F](k.suc)
            statement(k.suc)
        }
    }
    statement(Nat.0) and forall(k: Nat) {
        statement(k) implies statement(k.suc)
    }
    alt_induction(statement)
    forall(k: Nat) { statement(k) }
    statement(n)
}

/// A nonzero polynomial supported below `n` has fewer than `n` distinct roots
/// in any explicitly listed collection.
theorem polynomial_root_list_length_lt_support_bound[F: Field](
    p: Polynomial[F],
    roots: List[F],
    n: Nat
) {
    polynomial_support_bounded_by(p, n) and
        p != Polynomial[F].zero and
        roots.is_unique and
        polynomial_roots_on_list(p, roots)
    implies roots.length < n
} by {
    polynomial_root_list_bound_property_all[F](n)
    polynomial_root_list_bound_property[F](n) = forall(q: Polynomial[F], items: List[F]) {
        polynomial_support_bounded_by(q, n) and
            q != Polynomial[F].zero and
            items.is_unique and
            polynomial_roots_on_list(q, items)
        implies items.length < n
    }
}
