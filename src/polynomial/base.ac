/// Univariate polynomials as finitely supported coefficient functions.

from algebra.add import Add
from finite_set import FiniteSet
from algebra.module.finite_support import has_support_in, has_support_in_apply_zero, is_finitely_supported,
    pointwise_add_has_support_in_union, pointwise_neg_has_support_in,
    pointwise_neg_is_finitely_supported
from data.basic.function_algebra import pointwise_add, pointwise_neg
from data.basic.functions import function_extensionality
from nat import Nat
from algebra.one import One
from algebra.ring.ring import Ring
from semiring import Semiring
from data.basic.set import Set, empty_set_contains_eq, singleton_contains_eq, singleton_set_is_finite
from algebra.zero import Zero

/// A univariate polynomial over `R`, represented by finitely supported coefficients.
structure Polynomial[R: Semiring] {
    /// The coefficient of `X^n`.
    coeff: Nat -> R
} constraint {
    is_finitely_supported(coeff)
}

/// Polynomial extensionality from equality of coefficient functions.
theorem polynomial_ext[R: Semiring](p: Polynomial[R], q: Polynomial[R]) {
    p.coeff = q.coeff implies p = q
} by {
    if p.coeff = q.coeff {
        Polynomial[R].new(p.coeff) = Option.some(p)
        Polynomial[R].new(q.coeff) = Option.some(q)
        Polynomial[R].new(p.coeff) = Option.some(q)
        Option.some(p) = Option.some(q)
        p = q
    }
}

/// Polynomial extensionality from pointwise equality of coefficients.
theorem polynomial_ext_pointwise[R: Semiring](p: Polynomial[R], q: Polynomial[R]) {
    (forall(n: Nat) { p.coeff(n) = q.coeff(n) }) implies p = q
} by {
    if forall(n: Nat) { p.coeff(n) = q.coeff(n) } {
        function_extensionality(p.coeff, q.coeff)
        p.coeff = q.coeff
        polynomial_ext(p, q)
        p = q
    }
}

/// Equal polynomials have equal coefficient functions.
theorem polynomial_eq_coeff[R: Semiring](p: Polynomial[R], q: Polynomial[R]) {
    p = q implies p.coeff = q.coeff
}

/// Equal polynomials have equal coefficients at every exponent.
theorem polynomial_eq_coeff_at[R: Semiring](p: Polynomial[R], q: Polynomial[R], n: Nat) {
    p = q implies p.coeff(n) = q.coeff(n)
} by {
    p.coeff = q.coeff
    p.coeff(n) = q.coeff(n)
}

/// The singleton support at one exponent.
let polynomial_singleton_support(n: Nat) -> result: FiniteSet[Nat] satisfy {
    FiniteSet[Nat].new(Set[Nat].singleton(n)) = Option.some(result)
} by {
    singleton_set_is_finite(n)
}

/// The coefficient function of a monomial `r * X^n`.
define polynomial_monomial_coeff[R: Semiring](n: Nat, r: R, k: Nat) -> R {
    if k = n {
        r
    } else {
        R.0
    }
}

/// A monomial coefficient function is zero outside its singleton support.
theorem polynomial_monomial_coeff_zero_of_not_contains[R: Semiring](n: Nat, r: R, k: Nat) {
    not polynomial_singleton_support(n).contains(k) implies
    polynomial_monomial_coeff[R](n, r, k) = R.0
} by {
    FiniteSet[Nat].new(Set[Nat].singleton(n)) = Option.some(polynomial_singleton_support(n))
    polynomial_singleton_support(n).underlying_set = Set[Nat].singleton(n)
    polynomial_singleton_support(n).contains(k) = Set[Nat].singleton(n).contains(k)
    not Set[Nat].singleton(n).contains(k)
    singleton_contains_eq(n, k)
    Set[Nat].singleton(n).contains(k) = (n = k)
    n != k
    not n = k
    k != n
    not k = n
    polynomial_monomial_coeff[R](n, r, k) = R.0
}

/// A monomial coefficient function is supported at its exponent.
theorem polynomial_monomial_coeff_has_support[R: Semiring](n: Nat, r: R) {
    has_support_in(polynomial_monomial_coeff[R](n, r), polynomial_singleton_support(n))
} by {
    forall(k: Nat) {
        polynomial_monomial_coeff_zero_of_not_contains[R](n, r, k)
    }
    has_support_in(polynomial_monomial_coeff[R](n, r), polynomial_singleton_support(n)) = forall(k: Nat) {
        polynomial_singleton_support(n).contains(k) or polynomial_monomial_coeff[R](n, r, k) = R.0
    }
    has_support_in(polynomial_monomial_coeff[R](n, r), polynomial_singleton_support(n))
}

/// A monomial coefficient function is finitely supported.
theorem polynomial_monomial_coeff_is_finitely_supported[R: Semiring](n: Nat, r: R) {
    is_finitely_supported(polynomial_monomial_coeff[R](n, r))
} by {
    polynomial_monomial_coeff_has_support[R](n, r)
    exists(s: FiniteSet[Nat]) {
        s = polynomial_singleton_support(n) and has_support_in(polynomial_monomial_coeff[R](n, r), s)
    }
}

/// The monomial polynomial `r * X^n`.
let polynomial_monomial[R: Semiring](n: Nat, r: R) -> result: Polynomial[R] satisfy {
    Polynomial[R].new(polynomial_monomial_coeff[R](n, r)) = Option.some(result)
} by {
    polynomial_monomial_coeff_is_finitely_supported[R](n, r)
}

/// The constant polynomial with coefficient `r`.
define polynomial_constant[R: Semiring](r: R) -> Polynomial[R] {
    polynomial_monomial(Nat.0, r)
}

/// The zero polynomial.
let polynomial_zero[R: Semiring]: Polynomial[R] = polynomial_constant(R.0)

/// The one polynomial.
let polynomial_one[R: Semiring]: Polynomial[R] = polynomial_constant(R.1)

/// The coefficientwise sum of two polynomials.
let polynomial_add[R: Semiring](p: Polynomial[R], q: Polynomial[R]) -> result: Polynomial[R] satisfy {
    Polynomial[R].new(pointwise_add(p.coeff, q.coeff)) = Option.some(result)
} by {
    let s: FiniteSet[Nat] satisfy {
        has_support_in(p.coeff, s)
    }
    let u: FiniteSet[Nat] satisfy {
        has_support_in(q.coeff, u)
    }
    pointwise_add_has_support_in_union[Nat, R](p.coeff, q.coeff, s, u)
    is_finitely_supported(pointwise_add(p.coeff, q.coeff))
    has_support_in(pointwise_add(p.coeff, q.coeff), s.union(u))
    exists(v: FiniteSet[Nat]) {
        v = s.union(u) and has_support_in(pointwise_add(p.coeff, q.coeff), v)
    }
}

attributes Polynomial[R: Semiring] {
    /// Polynomial extensionality from equality of coefficients.
    let ext = polynomial_ext[R]

    /// The monomial `r * X^n`.
    let monomial: (Nat, R) -> Polynomial[R] = polynomial_monomial

    /// The constant polynomial with coefficient `r`.
    let constant: R -> Polynomial[R] = polynomial_constant

    /// The zero polynomial.
    let zero: Polynomial[R] = polynomial_zero[R]

    /// The one polynomial.
    let one: Polynomial[R] = polynomial_one[R]

    /// Coefficientwise addition of polynomials.
    define add(self, other: Polynomial[R]) -> Polynomial[R] {
        polynomial_add(self, other)
    }
}

/// Polynomials have coefficientwise addition.
instance Polynomial[R: Semiring]: Add {
    let add = Polynomial[R].add
}

/// Polynomials have a zero element.
instance Polynomial[R: Semiring]: Zero {
    let 0 = Polynomial[R].zero
}

/// Polynomials have the constant-one element.
instance Polynomial[R: Semiring]: One {
    let 1 = Polynomial[R].one
}

/// The zero polynomial has zero coefficient at every exponent.
theorem polynomial_zero_coeff[R: Semiring](n: Nat) {
    Polynomial[R].zero.coeff(n) = R.0
} by {
    Polynomial[R].zero = polynomial_zero[R]
    polynomial_zero[R] = polynomial_constant(R.0)
    polynomial_constant(R.0) = polynomial_monomial(Nat.0, R.0)
    Polynomial[R].new(polynomial_monomial_coeff[R](Nat.0, R.0)) =
        Option.some(polynomial_monomial(Nat.0, R.0))
    polynomial_monomial(Nat.0, R.0).coeff = polynomial_monomial_coeff[R](Nat.0, R.0)
    polynomial_monomial_coeff[R](Nat.0, R.0, n) = R.0
    polynomial_monomial(Nat.0, R.0).coeff(n) = R.0
    Polynomial[R].zero.coeff(n) = R.0
}

/// Addition of polynomials adds coefficients pointwise.
theorem polynomial_add_coeff[R: Semiring](p: Polynomial[R], q: Polynomial[R], n: Nat) {
    (p + q).coeff(n) = p.coeff(n) + q.coeff(n)
} by {
    Polynomial[R].new(pointwise_add(p.coeff, q.coeff)) = Option.some(polynomial_add(p, q))
    polynomial_add(p, q).coeff = pointwise_add(p.coeff, q.coeff)
    pointwise_add(p.coeff, q.coeff, n) = p.coeff(n) + q.coeff(n)
}

/// The monomial has the chosen coefficient at its exponent.
theorem polynomial_monomial_coeff_self[R: Semiring](n: Nat, r: R) {
    Polynomial[R].monomial(n, r).coeff(n) = r
} by {
    Polynomial[R].new(polynomial_monomial_coeff[R](n, r)) =
        Option.some(Polynomial[R].monomial(n, r))
    Polynomial[R].monomial(n, r).coeff = polynomial_monomial_coeff[R](n, r)
    polynomial_monomial_coeff[R](n, r, n) = r
}

/// The monomial is zero away from its exponent.
theorem polynomial_monomial_coeff_of_ne[R: Semiring](n: Nat, r: R, k: Nat) {
    k != n implies Polynomial[R].monomial(n, r).coeff(k) = R.0
} by {
    Polynomial[R].new(polynomial_monomial_coeff[R](n, r)) =
        Option.some(Polynomial[R].monomial(n, r))
    Polynomial[R].monomial(n, r).coeff = polynomial_monomial_coeff[R](n, r)
    polynomial_monomial_coeff[R](n, r, k) = R.0
    Polynomial[R].monomial(n, r).coeff(k) = R.0
}

/// A constant polynomial has its constant coefficient.
theorem polynomial_constant_coeff_zero[R: Semiring](r: R) {
    Polynomial[R].constant(r).coeff(Nat.0) = r
} by {
    Polynomial[R].constant(r) = polynomial_constant(r)
    polynomial_constant(r) = polynomial_monomial(Nat.0, r)
    Polynomial[R].monomial(Nat.0, r) = polynomial_monomial(Nat.0, r)
    polynomial_monomial_coeff_self(Nat.0, r)
    polynomial_monomial(Nat.0, r).coeff(Nat.0) = r
    Polynomial[R].constant(r).coeff(Nat.0) = r
}

/// A constant polynomial is zero at every nonzero exponent.
theorem polynomial_constant_coeff_of_ne_zero[R: Semiring](r: R, n: Nat) {
    n != Nat.0 implies Polynomial[R].constant(r).coeff(n) = R.0
} by {
    if n != Nat.0 {
        Polynomial[R].constant(r) = polynomial_constant(r)
        polynomial_constant(r) = polynomial_monomial(Nat.0, r)
        Polynomial[R].monomial(Nat.0, r) = polynomial_monomial(Nat.0, r)
        polynomial_monomial_coeff_of_ne(Nat.0, r, n)
        polynomial_monomial(Nat.0, r).coeff(n) = R.0
        Polynomial[R].constant(r).coeff(n) = R.0
    }
}

/// The one polynomial has constant coefficient one.
theorem polynomial_one_coeff_zero[R: Semiring] {
    Polynomial[R].one.coeff(Nat.0) = R.1
} by {
    Polynomial[R].one = polynomial_one[R]
    polynomial_one[R] = polynomial_constant(R.1)
    Polynomial[R].constant(R.1) = polynomial_constant(R.1)
    polynomial_constant_coeff_zero(R.1)
    polynomial_constant(R.1).coeff(Nat.0) = R.1
    Polynomial[R].one.coeff(Nat.0) = R.1
}

/// The one polynomial is zero at every nonzero exponent.
theorem polynomial_one_coeff_of_ne_zero[R: Semiring](n: Nat) {
    n != Nat.0 implies Polynomial[R].one.coeff(n) = R.0
} by {
    if n != Nat.0 {
        Polynomial[R].one = polynomial_one[R]
        polynomial_one[R] = polynomial_constant(R.1)
        Polynomial[R].constant(R.1) = polynomial_constant(R.1)
        polynomial_constant_coeff_of_ne_zero(R.1, n)
        polynomial_constant(R.1).coeff(n) = R.0
        Polynomial[R].one.coeff(n) = R.0
    }
}

/// The zero polynomial is supported in every finite set.
theorem polynomial_zero_has_support_in[R: Semiring](s: FiniteSet[Nat]) {
    has_support_in(Polynomial[R].zero.coeff, s)
} by {
    forall(n: Nat) {
        polynomial_zero_coeff[R](n)
    }
    has_support_in(Polynomial[R].zero.coeff, s) = forall(n: Nat) {
        s.contains(n) or Polynomial[R].zero.coeff(n) = R.0
    }
    has_support_in(Polynomial[R].zero.coeff, s)
}

/// The zero polynomial is supported in the empty finite set.
theorem polynomial_zero_has_support_empty[R: Semiring] {
    has_support_in(Polynomial[R].zero.coeff, FiniteSet[Nat].empty)
} by {
    polynomial_zero_has_support_in[R](FiniteSet[Nat].empty)
}

/// A polynomial supported in the empty finite set is the zero polynomial.
theorem polynomial_eq_zero_of_has_support_empty[R: Semiring](p: Polynomial[R]) {
    has_support_in(p.coeff, FiniteSet[Nat].empty) implies p = Polynomial[R].zero
} by {
    forall(n: Nat) {
        FiniteSet[Nat].empty.underlying_set = Set[Nat].empty_set
        FiniteSet[Nat].empty.contains(n) = Set[Nat].empty_set.contains(n)
        empty_set_contains_eq[Nat](n)
        not FiniteSet[Nat].empty.contains(n)
        has_support_in_apply_zero[Nat, R](p.coeff, FiniteSet[Nat].empty, n)
        p.coeff(n) = R.0
        polynomial_zero_coeff[R](n)
        p.coeff(n) = Polynomial[R].zero.coeff(n)
    }
    polynomial_ext_pointwise(p, Polynomial[R].zero)
}

/// A monomial is supported in its singleton support.
theorem polynomial_monomial_has_support_in_singleton[R: Semiring](n: Nat, r: R) {
    has_support_in(Polynomial[R].monomial(n, r).coeff, polynomial_singleton_support(n))
} by {
    Polynomial[R].new(polynomial_monomial_coeff[R](n, r)) =
        Option.some(Polynomial[R].monomial(n, r))
    Polynomial[R].monomial(n, r).coeff = polynomial_monomial_coeff[R](n, r)
    polynomial_monomial_coeff_has_support[R](n, r)
}

/// The sum of two polynomials is supported in the union of supports.
theorem polynomial_add_has_support_in_union[R: Semiring](
    p: Polynomial[R],
    q: Polynomial[R],
    s: FiniteSet[Nat],
    u: FiniteSet[Nat]
) {
    has_support_in(p.coeff, s) and has_support_in(q.coeff, u) implies
    has_support_in(polynomial_add(p, q).coeff, s.union(u))
} by {
    has_support_in(p.coeff, s)
    has_support_in(q.coeff, u)
    Polynomial[R].new(pointwise_add(p.coeff, q.coeff)) = Option.some(polynomial_add(p, q))
    (p + q).coeff = pointwise_add(p.coeff, q.coeff)
    pointwise_add_has_support_in_union[Nat, R](p.coeff, q.coeff, s, u)
    has_support_in(pointwise_add(p.coeff, q.coeff), s.union(u))
}

/// Addition of polynomials is associative.
theorem polynomial_add_assoc[R: Semiring](p: Polynomial[R], q: Polynomial[R], r: Polynomial[R]) {
    polynomial_add(p, polynomial_add(q, r)) = polynomial_add(polynomial_add(p, q), r)
} by {
    forall(n: Nat) {
        polynomial_add_coeff(q, r, n)
        polynomial_add_coeff(p, polynomial_add(q, r), n)
        polynomial_add_coeff(p, q, n)
        polynomial_add_coeff(polynomial_add(p, q), r, n)
        p.coeff(n) + (q.coeff(n) + r.coeff(n)) = (p.coeff(n) + q.coeff(n)) + r.coeff(n)
        polynomial_add(p, polynomial_add(q, r)).coeff(n) =
            polynomial_add(polynomial_add(p, q), r).coeff(n)
    }
    polynomial_ext_pointwise(polynomial_add(p, polynomial_add(q, r)), polynomial_add(polynomial_add(p, q), r))
}

/// Addition of polynomials is commutative.
theorem polynomial_add_comm[R: Semiring](p: Polynomial[R], q: Polynomial[R]) {
    polynomial_add(p, q) = polynomial_add(q, p)
} by {
    forall(n: Nat) {
        polynomial_add_coeff(p, q, n)
        polynomial_add_coeff(q, p, n)
        p.coeff(n) + q.coeff(n) = q.coeff(n) + p.coeff(n)
        polynomial_add(p, q).coeff(n) = polynomial_add(q, p).coeff(n)
    }
    polynomial_ext_pointwise(polynomial_add(p, q), polynomial_add(q, p))
}

/// Adding zero on the right changes no polynomial.
theorem polynomial_add_zero_right[R: Semiring](p: Polynomial[R]) {
    polynomial_add(p, polynomial_zero[R]) = p
} by {
    forall(n: Nat) {
        polynomial_add_coeff(p, polynomial_zero[R], n)
        polynomial_zero_coeff[R](n)
        p.coeff(n) + R.0 = p.coeff(n)
        polynomial_add(p, polynomial_zero[R]).coeff(n) = p.coeff(n)
    }
    polynomial_ext_pointwise(polynomial_add(p, polynomial_zero[R]), p)
}

/// Adding zero on the left changes no polynomial.
theorem polynomial_add_zero_left[R: Semiring](p: Polynomial[R]) {
    polynomial_add(polynomial_zero[R], p) = p
} by {
    forall(n: Nat) {
        polynomial_add_coeff(polynomial_zero[R], p, n)
        polynomial_zero_coeff[R](n)
        R.0 + p.coeff(n) = p.coeff(n)
        polynomial_add(polynomial_zero[R], p).coeff(n) = p.coeff(n)
    }
    polynomial_ext_pointwise(polynomial_add(polynomial_zero[R], p), p)
}

/// A polynomial is zero when all its coefficients are zero.
theorem polynomial_eq_zero_of_coeff_zero[R: Semiring](p: Polynomial[R]) {
    (forall(n: Nat) { p.coeff(n) = R.0 }) implies p = Polynomial[R].zero
} by {
    forall(n: Nat) {
        polynomial_zero_coeff[R](n)
        p.coeff(n) = Polynomial[R].zero.coeff(n)
    }
    polynomial_ext_pointwise(p, Polynomial[R].zero)
}

/// A zero polynomial has zero coefficients.
theorem polynomial_coeff_zero_of_eq_zero[R: Semiring](p: Polynomial[R], n: Nat) {
    p = Polynomial[R].zero implies p.coeff(n) = R.0
} by {
    p = Polynomial[R].zero
    polynomial_eq_coeff_at(p, Polynomial[R].zero, n)
    polynomial_zero_coeff[R](n)
}

/// The coefficientwise additive inverse of a polynomial.
let polynomial_neg[R: Ring](p: Polynomial[R]) -> result: Polynomial[R] satisfy {
    Polynomial[R].new(pointwise_neg(p.coeff)) = Option.some(result)
} by {
    is_finitely_supported(p.coeff)
    pointwise_neg_is_finitely_supported[Nat, R](p.coeff)
}

attributes Polynomial[R: Ring] {
    /// The coefficientwise additive inverse of a polynomial.
    define neg(self) -> Polynomial[R] {
        polynomial_neg(self)
    }

    /// The coefficientwise difference of two polynomials.
    define sub(self, other: Polynomial[R]) -> Polynomial[R] {
        self + other.neg
    }
}

/// Negation of a polynomial negates every coefficient.
theorem polynomial_neg_coeff[R: Ring](p: Polynomial[R], n: Nat) {
    p.neg.coeff(n) = -p.coeff(n)
} by {
    Polynomial[R].new(pointwise_neg(p.coeff)) = Option.some(polynomial_neg(p))
    polynomial_neg(p).coeff = pointwise_neg(p.coeff)
    pointwise_neg(p.coeff, n) = -p.coeff(n)
}

/// The coefficientwise difference subtracts coefficients.
theorem polynomial_sub_coeff[R: Ring](p: Polynomial[R], q: Polynomial[R], n: Nat) {
    p.sub(q).coeff(n) = p.coeff(n) + -q.coeff(n)
} by {
    polynomial_add_coeff(p, q.neg, n)
    polynomial_neg_coeff(q, n)
}

/// Negation preserves a specified support.
theorem polynomial_neg_has_support_in[R: Ring](p: Polynomial[R], s: FiniteSet[Nat]) {
    has_support_in(p.coeff, s) implies has_support_in(p.neg.coeff, s)
} by {
    Polynomial[R].new(pointwise_neg(p.coeff)) = Option.some(polynomial_neg(p))
    p.neg.coeff = pointwise_neg(p.coeff)
    pointwise_neg_has_support_in[Nat, R](p.coeff, s)
}

/// Subtraction is supported in the union of supports.
theorem polynomial_sub_has_support_in_union[R: Ring](
    p: Polynomial[R],
    q: Polynomial[R],
    s: FiniteSet[Nat],
    u: FiniteSet[Nat]
) {
    has_support_in(p.coeff, s) and has_support_in(q.coeff, u) implies
    has_support_in(p.sub(q).coeff, s.union(u))
} by {
    if has_support_in(p.coeff, s) and has_support_in(q.coeff, u) {
        polynomial_neg_has_support_in(q, u)
        has_support_in(q.neg.coeff, u)
        polynomial_add_has_support_in_union(p, q.neg, s, u)
        has_support_in(polynomial_add(p, q.neg).coeff, s.union(u))
        has_support_in(p.sub(q).coeff, s.union(u))
    }
}

/// Adding the additive inverse on the right gives zero.
theorem polynomial_add_neg_right[R: Ring](p: Polynomial[R]) {
    polynomial_add(p, p.neg) = Polynomial[R].zero
} by {
    forall(n: Nat) {
        polynomial_add_coeff(p, p.neg, n)
        polynomial_neg_coeff(p, n)
        p.coeff(n) + -p.coeff(n) = R.0
        polynomial_zero_coeff[R](n)
        (p + p.neg).coeff(n) = Polynomial[R].zero.coeff(n)
    }
    polynomial_ext_pointwise(p + p.neg, Polynomial[R].zero)
}

/// Subtracting a polynomial from itself gives zero.
theorem polynomial_sub_self[R: Ring](p: Polynomial[R]) {
    p.sub(p) = Polynomial[R].zero
} by {
    polynomial_add_neg_right(p)
}
