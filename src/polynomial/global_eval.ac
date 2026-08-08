/// Support-independent evaluation for finitely supported univariate polynomials.

from nat import Nat
from nat import lt_or_lte
from polynomial.eval import polynomial_support_bounded_by, polynomial_eval_bound,
    polynomial_support_bounded_by_monotone, polynomial_eval_bound_eq_of_support_bounded,
    polynomial_zero_support_bounded_by_zero, polynomial_constant_support_bounded_by_one,
    polynomial_one_support_bounded_by_one, polynomial_add_support_bounded_by,
    polynomial_eval_bound_zero, polynomial_eval_bound_constant_of_one_le,
    polynomial_eval_bound_one_of_one_le, polynomial_eval_bound_add,
    polynomial_support_bound_exists
from polynomial.base import Polynomial, polynomial_add_neg_right
from semiring import Semiring
from algebra.ring.ring import Ring
from algebra.add_group import left_cancel

numerals Nat

/// Support-independent polynomial evaluation.
let polynomial_eval[R: Semiring](p: Polynomial[R], x: R) -> result: R satisfy {
    forall(n: Nat) {
        polynomial_support_bounded_by(p, n) implies result = polynomial_eval_bound(p, x, n)
    }
} by {
    polynomial_support_bound_exists(p)
    let n: Nat satisfy {
        polynomial_support_bounded_by(p, n)
    }
    let value = polynomial_eval_bound(p, x, n)
    forall(m: Nat) {
        if polynomial_support_bounded_by(p, m) {
            polynomial_eval_bound_eq_of_support_bounded(p, x, n, m)
            polynomial_eval_bound(p, x, n) = polynomial_eval_bound(p, x, m)
            value = polynomial_eval_bound(p, x, m)
        }
    }
    let candidate = value
    forall(m: Nat) {
        if polynomial_support_bounded_by(p, m) {
            polynomial_eval_bound_eq_of_support_bounded(p, x, n, m)
            polynomial_eval_bound(p, x, n) = polynomial_eval_bound(p, x, m)
            candidate = polynomial_eval_bound(p, x, m)
        }
    }
    candidate = value
    forall(m: Nat) {
        polynomial_support_bounded_by(p, m) implies candidate = polynomial_eval_bound(p, x, m)
    }
    candidate = value and forall(m: Nat) {
        polynomial_support_bounded_by(p, m) implies candidate = polynomial_eval_bound(p, x, m)
    }
    exists(result0: R) {
        result0 = candidate and forall(m: Nat) {
            polynomial_support_bounded_by(p, m) implies result0 = polynomial_eval_bound(p, x, m)
        }
    }
}

/// Global evaluation agrees with bounded evaluation at any valid support bound.
theorem polynomial_eval_eq_eval_bound_of_support_bounded[R: Semiring](p: Polynomial[R], x: R, n: Nat) {
    polynomial_support_bounded_by(p, n) implies polynomial_eval(p, x) = polynomial_eval_bound(p, x, n)
} by {
    if polynomial_support_bounded_by(p, n) {
        polynomial_eval(p, x) = polynomial_eval_bound(p, x, n)
    }
}

/// Global evaluation of the zero polynomial is zero.
theorem polynomial_eval_zero[R: Semiring](x: R) {
    polynomial_eval(Polynomial[R].zero, x) = R.0
} by {
    polynomial_zero_support_bounded_by_zero[R]
    polynomial_eval_eq_eval_bound_of_support_bounded(Polynomial[R].zero, x, Nat.0)
    polynomial_eval_bound_zero(x, Nat.0)
    polynomial_eval(Polynomial[R].zero, x) = R.0
}

/// Global evaluation of a constant polynomial is its constant value.
theorem polynomial_eval_constant[R: Semiring](r: R, x: R) {
    polynomial_eval(Polynomial[R].constant(r), x) = r
} by {
    polynomial_constant_support_bounded_by_one(r)
    polynomial_eval_eq_eval_bound_of_support_bounded(Polynomial[R].constant(r), x, Nat.0.suc)
    polynomial_eval_bound_constant_of_one_le(r, x, Nat.0.suc)
    polynomial_eval(Polynomial[R].constant(r), x) = r
}

/// Global evaluation of the one polynomial is one.
theorem polynomial_eval_one[R: Semiring](x: R) {
    polynomial_eval(Polynomial[R].one, x) = R.1
} by {
    polynomial_one_support_bounded_by_one[R]
    polynomial_eval_eq_eval_bound_of_support_bounded(Polynomial[R].one, x, Nat.0.suc)
    polynomial_eval_bound_one_of_one_le(x, Nat.0.suc)
    polynomial_eval(Polynomial[R].one, x) = R.1
}

/// Global evaluation is additive.
theorem polynomial_eval_add[R: Semiring](p: Polynomial[R], q: Polynomial[R], x: R) {
    polynomial_eval(p + q, x) = polynomial_eval(p, x) + polynomial_eval(q, x)
} by {
    polynomial_support_bound_exists(p)
    let n: Nat satisfy {
        polynomial_support_bounded_by(p, n)
    }
    polynomial_support_bound_exists(q)
    let m: Nat satisfy {
        polynomial_support_bounded_by(q, m)
    }
    if n <= m {
        polynomial_support_bounded_by_monotone(p, n, m)
        polynomial_support_bounded_by(p, m)
        polynomial_add_support_bounded_by(p, q, m)
        polynomial_eval_eq_eval_bound_of_support_bounded(p + q, x, m)
        polynomial_eval_eq_eval_bound_of_support_bounded(p, x, m)
        polynomial_eval_eq_eval_bound_of_support_bounded(q, x, m)
        polynomial_eval_bound_add(p, q, x, m)
        polynomial_eval(p + q, x) = polynomial_eval_bound(p + q, x, m)
        polynomial_eval(p, x) = polynomial_eval_bound(p, x, m)
        polynomial_eval(q, x) = polynomial_eval_bound(q, x, m)
        polynomial_eval(p + q, x) = polynomial_eval(p, x) + polynomial_eval(q, x)
    } else {
        lt_or_lte(n, m)
        m <= n
        polynomial_support_bounded_by_monotone(q, m, n)
        polynomial_support_bounded_by(q, n)
        polynomial_add_support_bounded_by(p, q, n)
        polynomial_eval_eq_eval_bound_of_support_bounded(p + q, x, n)
        polynomial_eval_eq_eval_bound_of_support_bounded(p, x, n)
        polynomial_eval_eq_eval_bound_of_support_bounded(q, x, n)
        polynomial_eval_bound_add(p, q, x, n)
        polynomial_eval(p + q, x) = polynomial_eval_bound(p + q, x, n)
        polynomial_eval(p, x) = polynomial_eval_bound(p, x, n)
        polynomial_eval(q, x) = polynomial_eval_bound(q, x, n)
        polynomial_eval(p + q, x) = polynomial_eval(p, x) + polynomial_eval(q, x)
    }
}

/// Global evaluation sends coefficientwise negation to additive inverse.
theorem polynomial_eval_neg[R: Ring](p: Polynomial[R], x: R) {
    polynomial_eval(p.neg, x) = -polynomial_eval(p, x)
} by {
    polynomial_eval_add(p, p.neg, x)
    polynomial_eval(p + p.neg, x) = polynomial_eval(p, x) + polynomial_eval(p.neg, x)
    polynomial_add_neg_right(p)
    p + p.neg = Polynomial[R].zero
    polynomial_eval_zero(x)
    polynomial_eval(Polynomial[R].zero, x) = R.0
    polynomial_eval(p, x) + polynomial_eval(p.neg, x) = R.0
    polynomial_eval(p, x) + -polynomial_eval(p, x) = R.0
    polynomial_eval(p, x) + polynomial_eval(p.neg, x) = polynomial_eval(p, x) + -polynomial_eval(p, x)
    left_cancel(polynomial_eval(p, x), polynomial_eval(p.neg, x), -polynomial_eval(p, x))
    polynomial_eval(p.neg, x) = -polynomial_eval(p, x)
}

/// Global evaluation sends coefficientwise subtraction to subtraction of values.
theorem polynomial_eval_sub[R: Ring](p: Polynomial[R], q: Polynomial[R], x: R) {
    polynomial_eval(p.sub(q), x) = polynomial_eval(p, x) - polynomial_eval(q, x)
} by {
    p.sub(q) = p + q.neg
    polynomial_eval_add(p, q.neg, x)
    polynomial_eval(p + q.neg, x) = polynomial_eval(p, x) + polynomial_eval(q.neg, x)
    polynomial_eval_neg(q, x)
    polynomial_eval(q.neg, x) = -polynomial_eval(q, x)
    polynomial_eval(p.sub(q), x) = polynomial_eval(p, x) + -polynomial_eval(q, x)
    polynomial_eval(p, x) - polynomial_eval(q, x) = polynomial_eval(p, x) + -polynomial_eval(q, x)
    polynomial_eval(p.sub(q), x) = polynomial_eval(p, x) - polynomial_eval(q, x)
}
