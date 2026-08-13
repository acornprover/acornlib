/// The formal derivative of a polynomial and its multiplicity-theoretic
/// consequences: the derivative of a product with a linear factor, the
/// characterization of repeated roots as common roots of `p` and `p'`, and
/// squarefree polynomials as polynomials without repeated linear factors.
///
/// The derivative is defined coefficientwise: the coefficient of `X^n` in
/// `p'` is `(n + 1) * p_{n + 1}`.  Over any field a repeated root of `p`
/// (a root of multiplicity at least two) is a common root of `p` and `p'`,
/// and conversely a common root of `p` and `p'` is a repeated root; this
/// gives the classical characterization of squarefree polynomials as those
/// with no common root with their derivative.

from nat import Nat, from_nat, from_nat_add, from_nat_one, from_nat_zero, lt_suc, lte_trans,
    lte_suc_suc, lte_imp_not_lt, lt_trans, lt_or_lte, not_lt_zero, zero_or_suc, sub_zero,
    sub_self, suc_sub_one
from order import lt_imp_lte
from algebra.module.finite_support import has_support_in, is_finitely_supported
from semiring import Semiring
from comm_ring import CommRing
from algebra.field.field import Field, field_mul_eq_zero
from algebra.ring.ring import mul_neg_left
from finite_set import FiniteSet, fs_from_list
from list import range_contains_all_leq
from polynomial import Polynomial, polynomial_constant, polynomial_monomial, polynomial_eval,
    polynomial_eval_add, polynomial_eval_mul, polynomial_mul, polynomial_zero_coeff,
    polynomial_support_bound_exists, polynomial_add_coeff, polynomial_constant_coeff_of_ne_zero,
    polynomial_monomial_coeff_of_ne, polynomial_monomial_coeff_self, polynomial_ext_pointwise,
    polynomial_support_bounded_by, polynomial_support_bounded_by_apply,
    polynomial_support_bounded_by_monotone, polynomial_eval_constant
from polynomial.polynomial_vieta import linear_polynomial, linear_polynomial_eval,
    linear_polynomial_neg_root, linear_polynomial_coeff_zero, linear_polynomial_coeff_one
from polynomial.polynomial_multiplicity import polynomial_double_root,
    polynomial_double_root_implies_root
from polynomial.polynomial_multiplicity_deep import polynomial_mul_x_minus_a_coeff,
    polynomial_mul_x_minus_a_coeff_zero, polynomial_root_imp_linear_divides,
    polynomial_linear_divides_imp_root, polynomial_linear_divides_iff_root
from polynomial.polynomial_roots import polynomial_sub_eq_zero_iff
from polynomial.polynomial_divisibility import polynomial_divides, polynomial_divides_eval_witness
from polynomial_mul_assoc import polynomial_mul_assoc
from data.basic.set import list_set, list_set_contains_eq

numerals Nat

/// The coefficient of `X^n` in the formal derivative of `p`.
define polynomial_deriv_coeff[R: Semiring](p: Polynomial[R], n: Nat) -> R {
    from_nat[R](n.suc) * p.coeff(n.suc)
}

/// A derivative coefficient is zero beyond any support bound of the original
/// polynomial.
lemma derivative_coeff_zero_of_support_bound[R: Semiring](p: Polynomial[R], n: Nat, k: Nat) {
    polynomial_support_bounded_by(p, n) and not k < n implies
    polynomial_deriv_coeff(p, k) = R.0
} by {
    if polynomial_support_bounded_by(p, n) and not k < n {
        lt_or_lte(k, n)
        n <= k
        lte_suc_suc(n, k)
        n.suc <= k.suc
        lte_imp_not_lt(n, k.suc)
        not k.suc < n
        polynomial_support_bounded_by_apply(p, n, k.suc)
        p.coeff(k.suc) = R.0
        polynomial_deriv_coeff(p, k) = from_nat[R](k.suc) * p.coeff(k.suc)
        polynomial_deriv_coeff(p, k) = from_nat[R](k.suc) * R.0
        from_nat[R](k.suc) * R.0 = R.0
        polynomial_deriv_coeff(p, k) = R.0
    }
}

/// The derivative coefficient function is supported below any support bound
/// of the original polynomial.
theorem polynomial_deriv_coeff_has_support_in_range[R: Semiring](
    p: Polynomial[R], n: Nat
) {
    polynomial_support_bounded_by(p, n) implies
    has_support_in(polynomial_deriv_coeff(p), fs_from_list[Nat](n.range))
} by {
    if polynomial_support_bounded_by(p, n) {
        forall(k: Nat) {
            if not fs_from_list[Nat](n.range).contains(k) {
                fs_from_list[Nat](n.range).underlying_set = list_set(n.range)
                fs_from_list[Nat](n.range).contains(k) =
                    fs_from_list[Nat](n.range).underlying_set.contains(k)
                fs_from_list[Nat](n.range).underlying_set.contains(k) = list_set(n.range).contains(k)
                list_set_contains_eq(n.range, k)
                list_set(n.range).contains(k) = n.range.contains(k)
                not n.range.contains(k)
                range_contains_all_leq(n)
                if k < n {
                    n.range.contains(k)
                    false
                }
                not k < n
                derivative_coeff_zero_of_support_bound(p, n, k)
                polynomial_deriv_coeff(p, k) = R.0
            }
        }
        forall(k: Nat) {
            if fs_from_list[Nat](n.range).contains(k) {
                fs_from_list[Nat](n.range).contains(k) or polynomial_deriv_coeff(p, k) = R.0
            } else {
                not fs_from_list[Nat](n.range).contains(k)
                fs_from_list[Nat](n.range).underlying_set = list_set(n.range)
                fs_from_list[Nat](n.range).contains(k) =
                    fs_from_list[Nat](n.range).underlying_set.contains(k)
                fs_from_list[Nat](n.range).underlying_set.contains(k) = list_set(n.range).contains(k)
                list_set_contains_eq(n.range, k)
                list_set(n.range).contains(k) = n.range.contains(k)
                not n.range.contains(k)
                range_contains_all_leq(n)
                if k < n {
                    n.range.contains(k)
                    false
                }
                not k < n
                derivative_coeff_zero_of_support_bound(p, n, k)
                polynomial_deriv_coeff(p, k) = R.0
                fs_from_list[Nat](n.range).contains(k) or polynomial_deriv_coeff(p, k) = R.0
            }
        }
        has_support_in(polynomial_deriv_coeff(p), fs_from_list[Nat](n.range)) = forall(k: Nat) {
            fs_from_list[Nat](n.range).contains(k) or polynomial_deriv_coeff(p, k) = R.0
        }
        has_support_in(polynomial_deriv_coeff(p), fs_from_list[Nat](n.range))
    }
}

/// The derivative coefficient function is finitely supported.
theorem polynomial_deriv_coeff_is_finitely_supported[R: Semiring](p: Polynomial[R]) {
    is_finitely_supported(polynomial_deriv_coeff(p))
} by {
    polynomial_support_bound_exists(p)
    let n: Nat satisfy {
        polynomial_support_bounded_by(p, n)
    }
    polynomial_deriv_coeff_has_support_in_range(p, n)
    has_support_in(polynomial_deriv_coeff(p), fs_from_list[Nat](n.range))
    exists(s: FiniteSet[Nat]) {
        s = fs_from_list[Nat](n.range) and has_support_in(polynomial_deriv_coeff(p), s)
    }
    is_finitely_supported(polynomial_deriv_coeff(p))
}

/// The formal derivative of a polynomial.
let polynomial_deriv[R: Semiring](p: Polynomial[R]) -> result: Polynomial[R] satisfy {
    Polynomial[R].new(polynomial_deriv_coeff(p)) = Option.some(result)
} by {
    polynomial_deriv_coeff_is_finitely_supported(p)
}

/// The coefficient of `X^n` in the formal derivative.
theorem polynomial_deriv_coeff_apply[R: Semiring](p: Polynomial[R], n: Nat) {
    polynomial_deriv(p).coeff(n) = from_nat[R](n.suc) * p.coeff(n.suc)
} by {
    Polynomial[R].new(polynomial_deriv_coeff(p)) = Option.some(polynomial_deriv(p))
    polynomial_deriv(p).coeff = polynomial_deriv_coeff(p)
    polynomial_deriv(p).coeff(n) = polynomial_deriv_coeff(p, n)
    polynomial_deriv_coeff(p, n) = from_nat[R](n.suc) * p.coeff(n.suc)
    polynomial_deriv(p).coeff(n) = from_nat[R](n.suc) * p.coeff(n.suc)
}

/// The derivative of a constant polynomial is zero.
theorem polynomial_deriv_constant_zero[R: Semiring](r: R) {
    polynomial_deriv(polynomial_constant(r)) = Polynomial[R].zero
} by {
    forall(n: Nat) {
        polynomial_deriv_coeff_apply(polynomial_constant(r), n)
        polynomial_deriv(polynomial_constant(r)).coeff(n) = from_nat[R](n.suc) * polynomial_constant(r).coeff(n.suc)
        polynomial_constant_coeff_of_ne_zero(r, n.suc)
        polynomial_constant(r).coeff(n.suc) = R.0
        polynomial_deriv(polynomial_constant(r)).coeff(n) = from_nat[R](n.suc) * R.0
        from_nat[R](n.suc) * R.0 = R.0
        polynomial_deriv(polynomial_constant(r)).coeff(n) = R.0
        polynomial_zero_coeff[R](n)
        polynomial_deriv(polynomial_constant(r)).coeff(n) = Polynomial[R].zero.coeff(n)
    }
    polynomial_ext_pointwise(polynomial_deriv(polynomial_constant(r)), Polynomial[R].zero)
}

/// The derivative is additive.
theorem polynomial_deriv_add[R: Semiring](p: Polynomial[R], q: Polynomial[R]) {
    polynomial_deriv(p + q) = polynomial_deriv(p) + polynomial_deriv(q)
} by {
    forall(n: Nat) {
        polynomial_deriv_coeff_apply(p + q, n)
        polynomial_deriv(p + q).coeff(n) = from_nat[R](n.suc) * (p + q).coeff(n.suc)
        polynomial_add_coeff(p, q, n.suc)
        (p + q).coeff(n.suc) = p.coeff(n.suc) + q.coeff(n.suc)
        polynomial_deriv(p + q).coeff(n) = from_nat[R](n.suc) * (p.coeff(n.suc) + q.coeff(n.suc))
        from_nat[R](n.suc) * (p.coeff(n.suc) + q.coeff(n.suc)) =
            from_nat[R](n.suc) * p.coeff(n.suc) + from_nat[R](n.suc) * q.coeff(n.suc)
        polynomial_deriv(p + q).coeff(n) =
            from_nat[R](n.suc) * p.coeff(n.suc) + from_nat[R](n.suc) * q.coeff(n.suc)
        polynomial_deriv_coeff_apply(p, n)
        polynomial_deriv(p).coeff(n) = from_nat[R](n.suc) * p.coeff(n.suc)
        polynomial_deriv_coeff_apply(q, n)
        polynomial_deriv(q).coeff(n) = from_nat[R](n.suc) * q.coeff(n.suc)
        polynomial_add_coeff(polynomial_deriv(p), polynomial_deriv(q), n)
        (polynomial_deriv(p) + polynomial_deriv(q)).coeff(n) =
            polynomial_deriv(p).coeff(n) + polynomial_deriv(q).coeff(n)
        polynomial_deriv(p + q).coeff(n) = (polynomial_deriv(p) + polynomial_deriv(q)).coeff(n)
    }
    polynomial_ext_pointwise(polynomial_deriv(p + q), polynomial_deriv(p) + polynomial_deriv(q))
}

/// The derivative of a monomial of positive degree lowers the exponent by one
/// and multiplies by the exponent.
theorem polynomial_deriv_monomial[R: CommRing](n: Nat, r: R) {
    polynomial_deriv(polynomial_monomial(n.suc, r)) =
        polynomial_monomial(n, from_nat[R](n.suc) * r)
} by {
    forall(k: Nat) {
        polynomial_deriv_coeff_apply(polynomial_monomial(n.suc, r), k)
        polynomial_deriv(polynomial_monomial(n.suc, r)).coeff(k) =
            from_nat[R](k.suc) * polynomial_monomial(n.suc, r).coeff(k.suc)
        if k = n {
            polynomial_monomial_coeff_self(n.suc, r)
            polynomial_monomial(n.suc, r).coeff(n.suc) = r
            k = n
            polynomial_monomial(n.suc, r).coeff(k.suc) = r
            polynomial_deriv(polynomial_monomial(n.suc, r)).coeff(k) = from_nat[R](k.suc) * r
            polynomial_monomial_coeff_self(n, from_nat[R](n.suc) * r)
            polynomial_monomial(n, from_nat[R](n.suc) * r).coeff(n) = from_nat[R](n.suc) * r
            k = n
            polynomial_monomial(n, from_nat[R](n.suc) * r).coeff(k) = from_nat[R](n.suc) * r
            k.suc = n.suc
            from_nat[R](k.suc) = from_nat[R](n.suc)
            polynomial_deriv(polynomial_monomial(n.suc, r)).coeff(k) = from_nat[R](n.suc) * r
            polynomial_deriv(polynomial_monomial(n.suc, r)).coeff(k) =
                polynomial_monomial(n, from_nat[R](n.suc) * r).coeff(k)
        } else {
            k != n
            if k.suc = n.suc {
                k.suc = n.suc
                k = n
                false
            }
            k.suc != n.suc
            polynomial_monomial_coeff_of_ne[R](n.suc, r, k.suc)
            polynomial_monomial(n.suc, r).coeff(k.suc) = R.0
            polynomial_deriv(polynomial_monomial(n.suc, r)).coeff(k) = from_nat[R](k.suc) * R.0
            from_nat[R](k.suc) * R.0 = R.0
            polynomial_deriv(polynomial_monomial(n.suc, r)).coeff(k) = R.0
            polynomial_monomial_coeff_of_ne[R](n, from_nat[R](n.suc) * r, k)
            polynomial_monomial(n, from_nat[R](n.suc) * r).coeff(k) = R.0
            polynomial_deriv(polynomial_monomial(n.suc, r)).coeff(k) =
                polynomial_monomial(n, from_nat[R](n.suc) * r).coeff(k)
        }
    }
    polynomial_ext_pointwise(polynomial_deriv(polynomial_monomial(n.suc, r)),
        polynomial_monomial(n, from_nat[R](n.suc) * r))
}


/// The derivative of a linear polynomial is its leading coefficient.
theorem polynomial_deriv_linear[R: CommRing](a: R, b: R) {
    polynomial_deriv(linear_polynomial(a, b)) = polynomial_constant(a)
} by {
    polynomial_deriv_add(polynomial_constant(b), polynomial_monomial(Nat.1, a))
    polynomial_deriv(polynomial_constant(b) + polynomial_monomial(Nat.1, a)) =
        polynomial_deriv(polynomial_constant(b)) + polynomial_deriv(polynomial_monomial(Nat.1, a))
    polynomial_deriv_constant_zero(b)
    polynomial_deriv(polynomial_constant(b)) = Polynomial[R].zero
    polynomial_deriv_monomial(Nat.0, a)
    polynomial_deriv(polynomial_monomial(Nat.0.suc, a)) =
        polynomial_monomial(Nat.0, from_nat[R](Nat.0.suc) * a)
    linear_polynomial(a, b) = polynomial_constant(b) + polynomial_monomial(Nat.1, a)
    polynomial_deriv(linear_polynomial(a, b)) =
        Polynomial[R].zero + polynomial_monomial(Nat.0, from_nat[R](Nat.1) * a)
    Polynomial[R].zero + polynomial_monomial(Nat.0, from_nat[R](Nat.1) * a) =
        polynomial_monomial(Nat.0, from_nat[R](Nat.1) * a)
    from_nat_one[R]
    from_nat[R](Nat.1) = R.1
    from_nat[R](Nat.1) * a = R.1 * a
    R.1 * a = a
    from_nat[R](Nat.1) * a = a
    polynomial_monomial(Nat.0, from_nat[R](Nat.1) * a) = polynomial_monomial(Nat.0, a)
    polynomial_constant(a) = polynomial_monomial(Nat.0, a)
    polynomial_deriv(linear_polynomial(a, b)) = polynomial_constant(a)
}

/// The derivative of `X - a` is the constant one.
theorem polynomial_deriv_linear_factor[R: CommRing](a: R) {
    polynomial_deriv(linear_polynomial(R.1, -a)) = polynomial_constant(R.1)
} by {
    polynomial_deriv_linear(R.1, -a)
}

/// The single-factor product rule: the derivative of `(X - a) * r` is
/// `r + (X - a) * r'`.
theorem polynomial_deriv_x_minus_a_mul[R: CommRing](a: R, r: Polynomial[R]) {
    polynomial_deriv(polynomial_mul(linear_polynomial(R.1, -a), r)) =
        r + polynomial_mul(linear_polynomial(R.1, -a), polynomial_deriv(r))
} by {
    forall(n: Nat) {
        polynomial_deriv_coeff_apply(polynomial_mul(linear_polynomial(R.1, -a), r), n)
        polynomial_deriv(polynomial_mul(linear_polynomial(R.1, -a), r)).coeff(n) =
            from_nat[R](n.suc) * polynomial_mul(linear_polynomial(R.1, -a), r).coeff(n.suc)
        if n = Nat.0 {
            polynomial_mul_x_minus_a_coeff(a, r, Nat.1)
            polynomial_mul(linear_polynomial(R.1, -a), r).coeff(Nat.1) =
                (-a) * r.coeff(Nat.1) + r.coeff(Nat.1 - Nat.1)
            sub_self(Nat.1)
            Nat.1 - Nat.1 = Nat.0
            polynomial_mul(linear_polynomial(R.1, -a), r).coeff(Nat.1) =
                (-a) * r.coeff(Nat.1) + r.coeff(Nat.0)
            n = Nat.0
            polynomial_deriv(polynomial_mul(linear_polynomial(R.1, -a), r)).coeff(n) =
                from_nat[R](Nat.1) * ((-a) * r.coeff(Nat.1) + r.coeff(Nat.0))
            from_nat_one[R]
            from_nat[R](Nat.1) = R.1
            from_nat[R](Nat.1) * ((-a) * r.coeff(Nat.1) + r.coeff(Nat.0)) =
                (-a) * r.coeff(Nat.1) + r.coeff(Nat.0)
            polynomial_deriv(polynomial_mul(linear_polynomial(R.1, -a), r)).coeff(n) =
                (-a) * r.coeff(Nat.1) + r.coeff(Nat.0)
            polynomial_deriv_coeff_apply(r, Nat.0)
            polynomial_deriv(r).coeff(Nat.0) = from_nat[R](Nat.1) * r.coeff(Nat.1)
            from_nat_one[R]
            from_nat[R](Nat.1) = R.1
            polynomial_deriv(r).coeff(Nat.0) = R.1 * r.coeff(Nat.1)
            R.1 * r.coeff(Nat.1) = r.coeff(Nat.1)
            polynomial_deriv(r).coeff(Nat.0) = r.coeff(Nat.1)
            polynomial_mul_x_minus_a_coeff_zero(a, polynomial_deriv(r))
            polynomial_mul(linear_polynomial(R.1, -a), polynomial_deriv(r)).coeff(Nat.0) =
                (-a) * polynomial_deriv(r).coeff(Nat.0)
            polynomial_mul(linear_polynomial(R.1, -a), polynomial_deriv(r)).coeff(Nat.0) =
                (-a) * r.coeff(Nat.1)
            polynomial_add_coeff(r, polynomial_mul(linear_polynomial(R.1, -a), polynomial_deriv(r)), Nat.0)
            (r + polynomial_mul(linear_polynomial(R.1, -a), polynomial_deriv(r))).coeff(Nat.0) =
                r.coeff(Nat.0) + polynomial_mul(linear_polynomial(R.1, -a), polynomial_deriv(r)).coeff(Nat.0)
            (r + polynomial_mul(linear_polynomial(R.1, -a), polynomial_deriv(r))).coeff(Nat.0) =
                r.coeff(Nat.0) + (-a) * r.coeff(Nat.1)
            polynomial_deriv(polynomial_mul(linear_polynomial(R.1, -a), r)).coeff(n) =
                (r + polynomial_mul(linear_polynomial(R.1, -a), polynomial_deriv(r))).coeff(n)
        } else {
            n != Nat.0
            zero_or_suc(n)
            let np: Nat satisfy {
                n = np.suc
            }
            n = np.suc
            lte_suc_suc(Nat.0, n)
            Nat.1 <= n.suc
            polynomial_mul_x_minus_a_coeff(a, r, n.suc)
            polynomial_mul(linear_polynomial(R.1, -a), r).coeff(n.suc) =
                (-a) * r.coeff(n.suc) + r.coeff(n.suc - Nat.1)
            suc_sub_one(n)
            n.suc - Nat.1 = n
            polynomial_mul(linear_polynomial(R.1, -a), r).coeff(n.suc) =
                (-a) * r.coeff(n.suc) + r.coeff(n)
            polynomial_deriv(polynomial_mul(linear_polynomial(R.1, -a), r)).coeff(n) =
                from_nat[R](n.suc) * ((-a) * r.coeff(n.suc) + r.coeff(n))
            from_nat[R](n.suc) * ((-a) * r.coeff(n.suc) + r.coeff(n)) =
                from_nat[R](n.suc) * ((-a) * r.coeff(n.suc)) + from_nat[R](n.suc) * r.coeff(n)
            polynomial_deriv(polynomial_mul(linear_polynomial(R.1, -a), r)).coeff(n) =
                from_nat[R](n.suc) * ((-a) * r.coeff(n.suc)) + from_nat[R](n.suc) * r.coeff(n)
            from_nat_add[R](n, Nat.1)
            from_nat[R](n + Nat.1) = from_nat[R](n) + from_nat[R](Nat.1)
            from_nat_one[R]
            from_nat[R](Nat.1) = R.1
            n + Nat.1 = n.suc
            from_nat[R](n.suc) = from_nat[R](n) + R.1
            from_nat[R](n.suc) * r.coeff(n) = (from_nat[R](n) + R.1) * r.coeff(n)
            (from_nat[R](n) + R.1) * r.coeff(n) = from_nat[R](n) * r.coeff(n) + r.coeff(n)
            from_nat[R](n.suc) * r.coeff(n) = from_nat[R](n) * r.coeff(n) + r.coeff(n)
            polynomial_deriv(polynomial_mul(linear_polynomial(R.1, -a), r)).coeff(n) =
                from_nat[R](n.suc) * ((-a) * r.coeff(n.suc)) +
                (from_nat[R](n) * r.coeff(n) + r.coeff(n))
            polynomial_deriv(polynomial_mul(linear_polynomial(R.1, -a), r)).coeff(n) =
                (r.coeff(n) + from_nat[R](n) * r.coeff(n)) +
                from_nat[R](n.suc) * ((-a) * r.coeff(n.suc))
            polynomial_deriv_coeff_apply(r, n)
            polynomial_deriv(r).coeff(n) = from_nat[R](n.suc) * r.coeff(n.suc)
            polynomial_mul_x_minus_a_coeff(a, polynomial_deriv(r), n)
            polynomial_mul(linear_polynomial(R.1, -a), polynomial_deriv(r)).coeff(n) =
                (-a) * polynomial_deriv(r).coeff(n) + polynomial_deriv(r).coeff(n - Nat.1)
            polynomial_deriv_coeff_apply(r, n - Nat.1)
            polynomial_deriv(r).coeff(n - Nat.1) = from_nat[R]((n - Nat.1).suc) * r.coeff((n - Nat.1).suc)
            suc_sub_one(np)
            np.suc - Nat.1 = np
            n = np.suc
            n - Nat.1 = np
            (n - Nat.1).suc = n
            polynomial_deriv(r).coeff(n - Nat.1) = from_nat[R](n) * r.coeff(n)
            polynomial_mul(linear_polynomial(R.1, -a), polynomial_deriv(r)).coeff(n) =
                (-a) * (from_nat[R](n.suc) * r.coeff(n.suc)) + from_nat[R](n) * r.coeff(n)
            polynomial_add_coeff(r, polynomial_mul(linear_polynomial(R.1, -a), polynomial_deriv(r)), n)
            (r + polynomial_mul(linear_polynomial(R.1, -a), polynomial_deriv(r))).coeff(n) =
                r.coeff(n) + polynomial_mul(linear_polynomial(R.1, -a), polynomial_deriv(r)).coeff(n)
            (r + polynomial_mul(linear_polynomial(R.1, -a), polynomial_deriv(r))).coeff(n) =
                r.coeff(n) + ((-a) * (from_nat[R](n.suc) * r.coeff(n.suc)) +
                    from_nat[R](n) * r.coeff(n))
            r.coeff(n) + ((-a) * (from_nat[R](n.suc) * r.coeff(n.suc)) +
                from_nat[R](n) * r.coeff(n)) =
                (r.coeff(n) + from_nat[R](n) * r.coeff(n)) +
                (-a) * (from_nat[R](n.suc) * r.coeff(n.suc))
            polynomial_deriv(polynomial_mul(linear_polynomial(R.1, -a), r)).coeff(n) =
                (r.coeff(n) + from_nat[R](n) * r.coeff(n)) +
                (-a) * (from_nat[R](n.suc) * r.coeff(n.suc))
            from_nat[R](n.suc) * ((-a) * r.coeff(n.suc)) =
                (-a) * (from_nat[R](n.suc) * r.coeff(n.suc))
            (r.coeff(n) + from_nat[R](n) * r.coeff(n)) +
                from_nat[R](n.suc) * ((-a) * r.coeff(n.suc)) =
                (r.coeff(n) + from_nat[R](n) * r.coeff(n)) +
                (-a) * (from_nat[R](n.suc) * r.coeff(n.suc))
            polynomial_deriv(polynomial_mul(linear_polynomial(R.1, -a), r)).coeff(n) =
                (r + polynomial_mul(linear_polynomial(R.1, -a), polynomial_deriv(r))).coeff(n)
        }
    }
    polynomial_ext_pointwise(polynomial_deriv(polynomial_mul(linear_polynomial(R.1, -a), r)),
        r + polynomial_mul(linear_polynomial(R.1, -a), polynomial_deriv(r)))
}

// ---------------------------------------------------------------------------
// Repeated roots and the derivative
// ---------------------------------------------------------------------------

/// A repeated root of `p` is a root of the derivative of `p`.
theorem polynomial_double_root_imp_deriv_root[R: CommRing](p: Polynomial[R], a: R) {
    polynomial_double_root(p, a) implies polynomial_eval(polynomial_deriv(p), a) = R.0
} by {
    if polynomial_double_root(p, a) {
        let q: Polynomial[R] satisfy {
            polynomial_mul(polynomial_mul(linear_polynomial(R.1, -a),
                linear_polynomial(R.1, -a)), q) = p
        }
        polynomial_mul_assoc(linear_polynomial(R.1, -a), linear_polynomial(R.1, -a), q)
        polynomial_mul(polynomial_mul(linear_polynomial(R.1, -a), linear_polynomial(R.1, -a)), q) =
            polynomial_mul(linear_polynomial(R.1, -a),
                polynomial_mul(linear_polynomial(R.1, -a), q))
        polynomial_mul(polynomial_mul(linear_polynomial(R.1, -a), linear_polynomial(R.1, -a)), q) = p
        p = polynomial_mul(linear_polynomial(R.1, -a),
            polynomial_mul(linear_polynomial(R.1, -a), q))
        polynomial_deriv_x_minus_a_mul(a, polynomial_mul(linear_polynomial(R.1, -a), q))
        polynomial_deriv(polynomial_mul(linear_polynomial(R.1, -a),
            polynomial_mul(linear_polynomial(R.1, -a), q))) =
            polynomial_mul(linear_polynomial(R.1, -a), q) +
            polynomial_mul(linear_polynomial(R.1, -a),
                polynomial_deriv(polynomial_mul(linear_polynomial(R.1, -a), q)))
        polynomial_deriv(p) =
            polynomial_mul(linear_polynomial(R.1, -a), q) +
            polynomial_mul(linear_polynomial(R.1, -a),
                polynomial_deriv(polynomial_mul(linear_polynomial(R.1, -a), q)))
        polynomial_eval_add(polynomial_mul(linear_polynomial(R.1, -a), q),
            polynomial_mul(linear_polynomial(R.1, -a),
                polynomial_deriv(polynomial_mul(linear_polynomial(R.1, -a), q))), a)
        polynomial_eval(polynomial_deriv(p), a) =
            polynomial_eval(polynomial_mul(linear_polynomial(R.1, -a), q), a) +
            polynomial_eval(polynomial_mul(linear_polynomial(R.1, -a),
                polynomial_deriv(polynomial_mul(linear_polynomial(R.1, -a), q))), a)
        polynomial_eval_mul(linear_polynomial(R.1, -a), q, a)
        polynomial_eval(polynomial_mul(linear_polynomial(R.1, -a), q), a) =
            polynomial_eval(linear_polynomial(R.1, -a), a) * polynomial_eval(q, a)
        linear_polynomial_neg_root(a)
        polynomial_eval(linear_polynomial(R.1, -a), a) = R.0
        polynomial_eval(polynomial_mul(linear_polynomial(R.1, -a), q), a) = R.0 * polynomial_eval(q, a)
        R.0 * polynomial_eval(q, a) = R.0
        polynomial_eval(polynomial_mul(linear_polynomial(R.1, -a), q), a) = R.0
        polynomial_eval_mul(linear_polynomial(R.1, -a),
            polynomial_deriv(polynomial_mul(linear_polynomial(R.1, -a), q)), a)
        polynomial_eval(polynomial_mul(linear_polynomial(R.1, -a),
            polynomial_deriv(polynomial_mul(linear_polynomial(R.1, -a), q))), a) =
            polynomial_eval(linear_polynomial(R.1, -a), a) *
            polynomial_eval(polynomial_deriv(polynomial_mul(linear_polynomial(R.1, -a), q)), a)
        polynomial_eval(polynomial_mul(linear_polynomial(R.1, -a),
            polynomial_deriv(polynomial_mul(linear_polynomial(R.1, -a), q))), a) =
            R.0 * polynomial_eval(polynomial_deriv(polynomial_mul(linear_polynomial(R.1, -a), q)), a)
        R.0 * polynomial_eval(polynomial_deriv(polynomial_mul(linear_polynomial(R.1, -a), q)), a) = R.0
        polynomial_eval(polynomial_mul(linear_polynomial(R.1, -a),
            polynomial_deriv(polynomial_mul(linear_polynomial(R.1, -a), q))), a) = R.0
        polynomial_eval(polynomial_deriv(p), a) = R.0 + R.0
        R.0 + R.0 = R.0
        polynomial_eval(polynomial_deriv(p), a) = R.0
    }
}

/// A common root of `p` and its derivative is a repeated root of `p`.
theorem polynomial_root_and_deriv_root_imp_double_root[R: CommRing](p: Polynomial[R], a: R) {
    polynomial_eval(p, a) = R.0 and polynomial_eval(polynomial_deriv(p), a) = R.0 implies
    polynomial_double_root(p, a)
} by {
    if polynomial_eval(p, a) = R.0 and polynomial_eval(polynomial_deriv(p), a) = R.0 {
        polynomial_root_imp_linear_divides(p, a)
        polynomial_divides(linear_polynomial(R.1, -a), p)
        let q: Polynomial[R] satisfy {
            polynomial_mul(linear_polynomial(R.1, -a), q) = p
        }
        polynomial_deriv_x_minus_a_mul(a, q)
        polynomial_deriv(polynomial_mul(linear_polynomial(R.1, -a), q)) =
            q + polynomial_mul(linear_polynomial(R.1, -a), polynomial_deriv(q))
        polynomial_mul(linear_polynomial(R.1, -a), q) = p
        polynomial_deriv(p) = q + polynomial_mul(linear_polynomial(R.1, -a), polynomial_deriv(q))
        polynomial_eval_add(q, polynomial_mul(linear_polynomial(R.1, -a), polynomial_deriv(q)), a)
        polynomial_eval(polynomial_deriv(p), a) =
            polynomial_eval(q, a) + polynomial_eval(polynomial_mul(linear_polynomial(R.1, -a),
                polynomial_deriv(q)), a)
        polynomial_eval(polynomial_deriv(p), a) = R.0
        polynomial_eval_mul(linear_polynomial(R.1, -a), polynomial_deriv(q), a)
        polynomial_eval(polynomial_mul(linear_polynomial(R.1, -a), polynomial_deriv(q)), a) =
            polynomial_eval(linear_polynomial(R.1, -a), a) * polynomial_eval(polynomial_deriv(q), a)
        linear_polynomial_neg_root(a)
        polynomial_eval(linear_polynomial(R.1, -a), a) = R.0
        polynomial_eval(polynomial_mul(linear_polynomial(R.1, -a), polynomial_deriv(q)), a) =
            R.0 * polynomial_eval(polynomial_deriv(q), a)
        R.0 * polynomial_eval(polynomial_deriv(q), a) = R.0
        polynomial_eval(polynomial_mul(linear_polynomial(R.1, -a), polynomial_deriv(q)), a) = R.0
        polynomial_eval(q, a) + R.0 = R.0
        polynomial_eval(q, a) = R.0
        polynomial_root_imp_linear_divides(q, a)
        polynomial_divides(linear_polynomial(R.1, -a), q)
        let r: Polynomial[R] satisfy {
            polynomial_mul(linear_polynomial(R.1, -a), r) = q
        }
        polynomial_mul_assoc(linear_polynomial(R.1, -a), linear_polynomial(R.1, -a), r)
        polynomial_mul(polynomial_mul(linear_polynomial(R.1, -a), linear_polynomial(R.1, -a)), r) =
            polynomial_mul(linear_polynomial(R.1, -a),
                polynomial_mul(linear_polynomial(R.1, -a), r))
        polynomial_mul(linear_polynomial(R.1, -a), r) = q
        polynomial_mul(linear_polynomial(R.1, -a), q) = p
        polynomial_mul(polynomial_mul(linear_polynomial(R.1, -a), linear_polynomial(R.1, -a)), r) = p
        exists(s: Polynomial[R]) {
            s = r and polynomial_mul(polynomial_mul(linear_polynomial(R.1, -a),
                linear_polynomial(R.1, -a)), s) = p
        }
        polynomial_divides(polynomial_mul(linear_polynomial(R.1, -a), linear_polynomial(R.1, -a)), p)
        polynomial_double_root(p, a)
    }
}

// ---------------------------------------------------------------------------
// Squarefree polynomials
// ---------------------------------------------------------------------------

/// A polynomial is squarefree when no linear factor square divides it, i.e.
/// when it has no repeated root.
define polynomial_squarefree[R: CommRing](p: Polynomial[R]) -> Bool {
    forall(a: R) {
        not polynomial_double_root(p, a)
    }
}

/// A polynomial is squarefree exactly when it has no common root with its
/// derivative.
theorem polynomial_squarefree_iff_no_common_root_with_deriv[R: CommRing](p: Polynomial[R]) {
    polynomial_squarefree(p) = (not exists(a: R) {
        polynomial_eval(p, a) = R.0 and polynomial_eval(polynomial_deriv(p), a) = R.0
    })
} by {
    if polynomial_squarefree(p) {
        if exists(a: R) {
            polynomial_eval(p, a) = R.0 and polynomial_eval(polynomial_deriv(p), a) = R.0
        } {
            let (a: R) satisfy {
                polynomial_eval(p, a) = R.0 and polynomial_eval(polynomial_deriv(p), a) = R.0
            }
            polynomial_root_and_deriv_root_imp_double_root(p, a)
            polynomial_double_root(p, a)
            polynomial_squarefree(p) = forall(x: R) {
                not polynomial_double_root(p, x)
            }
            not polynomial_double_root(p, a)
            false
        }
        not exists(a: R) {
            polynomial_eval(p, a) = R.0 and polynomial_eval(polynomial_deriv(p), a) = R.0
        }
    }
    if not exists(a: R) {
        polynomial_eval(p, a) = R.0 and polynomial_eval(polynomial_deriv(p), a) = R.0
    } {
        forall(a: R) {
            if polynomial_double_root(p, a) {
                polynomial_double_root_implies_root(p, a)
                polynomial_eval(p, a) = R.0
                polynomial_double_root_imp_deriv_root(p, a)
                polynomial_eval(polynomial_deriv(p), a) = R.0
                exists(x: R) {
                    polynomial_eval(p, x) = R.0 and polynomial_eval(polynomial_deriv(p), x) = R.0
                }
                not exists(x: R) {
                    polynomial_eval(p, x) = R.0 and polynomial_eval(polynomial_deriv(p), x) = R.0
                }
                false
            }
            not polynomial_double_root(p, a)
        }
        polynomial_squarefree(p) = forall(x: R) {
            not polynomial_double_root(p, x)
        }
        polynomial_squarefree(p)
    }
}
