/// Vieta's formulas for cubic polynomials: the product of the three monic
/// linear factors `X - r1`, `X - r2`, `X - r3` is the monic cubic
/// `X^3 - (r1 + r2 + r3) X^2 + (r1 r2 + r1 r3 + r2 r3) X - r1 r2 r3`, whose
/// roots are exactly `r1`, `r2` and `r3`.

from nat import Nat, pow_one, pow_add, lt_suc
from order import lt_imp_ne, lt_trans
from semiring import Semiring
from comm_ring import CommRing
from algebra.add_group import inverse_add
from algebra.ring.ring import mul_neg_left, mul_neg_right
from polynomial import Polynomial, polynomial_constant, polynomial_monomial,
    polynomial_monomial_coeff_self, polynomial_monomial_coeff_of_ne,
    polynomial_constant_coeff_zero, polynomial_constant_coeff_of_ne_zero,
    polynomial_ext_pointwise, polynomial_add_coeff, polynomial_add_assoc,
    polynomial_eval, polynomial_eval_constant, polynomial_eval_add,
    polynomial_eval_mul, polynomial_mul
from polynomial.polynomial_eval_deep import polynomial_eval_monomial
from polynomial.polynomial_monomial import polynomial_monomial_mul,
    polynomial_constant_mul_monomial, polynomial_monomial_mul_constant
from polynomial.polynomial_ring_theory import polynomial_constant_mul, polynomial_mul_add_right
from polynomial_mul_distrib import polynomial_mul_add_left
from polynomial.polynomial_vieta import linear_polynomial, quadratic_polynomial,
    polynomial_monomial_add_same, polynomial_linear_mul_quadratic,
    linear_polynomial_eval, linear_polynomial_neg_root, polynomial_root_mul_left,
    polynomial_root_mul_right

numerals Nat

/// The cubic polynomial `a * X^3 + b * X^2 + c * X + d`.
define cubic_polynomial[R: Semiring](a: R, b: R, c: R, d: R) -> Polynomial[R] {
    polynomial_constant(d) + polynomial_monomial(Nat.1, c) +
    polynomial_monomial(Nat.2, b) + polynomial_monomial(Nat.3, a)
}

/// The second power of an element is its square.
lemma cubic_pow_two[R: Semiring](x: R) {
    x.pow(Nat.2) = x * x
} by {
    pow_add(x, Nat.1, Nat.1)
    x.pow(Nat.1) * x.pow(Nat.1) = x.pow(Nat.1 + Nat.1)
    Nat.1 + Nat.1 = Nat.2
    x.pow(Nat.1) * x.pow(Nat.1) = x.pow(Nat.2)
    pow_one(x)
    x.pow(Nat.1) = x
    x * x = x.pow(Nat.2)
    x.pow(Nat.2) = x * x
}

/// The third power of an element is its cube.
lemma cubic_pow_three[R: Semiring](x: R) {
    x.pow(Nat.3) = x * x * x
} by {
    pow_add(x, Nat.2, Nat.1)
    x.pow(Nat.2) * x.pow(Nat.1) = x.pow(Nat.2 + Nat.1)
    Nat.2 + Nat.1 = Nat.3
    x.pow(Nat.2) * x.pow(Nat.1) = x.pow(Nat.3)
    cubic_pow_two(x)
    x.pow(Nat.2) = x * x
    (x * x) * x.pow(Nat.1) = x.pow(Nat.3)
    pow_one(x)
    x.pow(Nat.1) = x
    (x * x) * x = x.pow(Nat.3)
    (x * x) * x = x * x * x
    x.pow(Nat.3) = x * x * x
}

/// The product of a quadratic and a linear polynomial expands into six
/// The product of the left part of a quadratic with a linear polynomial:
/// (c + b X) (d X + e) expands with the two linear monomials merged.
lemma quadratic_mul_linear_left_part[R: CommRing](b: R, c: R, d: R, e: R) {
    polynomial_mul(polynomial_constant(c) + polynomial_monomial(Nat.1, b),
        linear_polynomial(d, e)) =
        polynomial_constant(c * e) + polynomial_monomial(Nat.1, c * d + b * e) +
        polynomial_monomial(Nat.2, b * d)
} by {
    polynomial_mul_add_right(polynomial_constant(c), polynomial_monomial(Nat.1, b),
        linear_polynomial(d, e))
    polynomial_mul(polynomial_constant(c) + polynomial_monomial(Nat.1, b),
        linear_polynomial(d, e)) =
        polynomial_mul(polynomial_constant(c), linear_polynomial(d, e)) +
        polynomial_mul(polynomial_monomial(Nat.1, b), linear_polynomial(d, e))
    polynomial_mul_add_left(polynomial_constant(c), polynomial_constant(e),
        polynomial_monomial(Nat.1, d))
    polynomial_mul(polynomial_constant(c), polynomial_constant(e) +
        polynomial_monomial(Nat.1, d)) =
        polynomial_mul(polynomial_constant(c), polynomial_constant(e)) +
        polynomial_mul(polynomial_constant(c), polynomial_monomial(Nat.1, d))
    polynomial_mul_add_left(polynomial_monomial(Nat.1, b), polynomial_constant(e),
        polynomial_monomial(Nat.1, d))
    polynomial_mul(polynomial_monomial(Nat.1, b), polynomial_constant(e) +
        polynomial_monomial(Nat.1, d)) =
        polynomial_mul(polynomial_monomial(Nat.1, b), polynomial_constant(e)) +
        polynomial_mul(polynomial_monomial(Nat.1, b), polynomial_monomial(Nat.1, d))
    polynomial_constant_mul(c, e)
    polynomial_mul(polynomial_constant(c), polynomial_constant(e)) =
        polynomial_constant(c * e)
    polynomial_constant_mul_monomial(c, Nat.1, d)
    polynomial_mul(polynomial_constant(c), polynomial_monomial(Nat.1, d)) =
        polynomial_monomial(Nat.1, c * d)
    polynomial_monomial_mul_constant(Nat.1, b, e)
    polynomial_mul(polynomial_monomial(Nat.1, b), polynomial_constant(e)) =
        polynomial_monomial(Nat.1, b * e)
    polynomial_monomial_mul(Nat.1, b, Nat.1, d)
    polynomial_mul(polynomial_monomial(Nat.1, b), polynomial_monomial(Nat.1, d)) =
        polynomial_monomial(Nat.1 + Nat.1, b * d)
    Nat.1 + Nat.1 = Nat.2
    polynomial_mul(polynomial_monomial(Nat.1, b), polynomial_monomial(Nat.1, d)) =
        polynomial_monomial(Nat.2, b * d)
    polynomial_mul(polynomial_constant(c), linear_polynomial(d, e)) =
        polynomial_constant(c * e) + polynomial_monomial(Nat.1, c * d)
    polynomial_mul(polynomial_monomial(Nat.1, b), linear_polynomial(d, e)) =
        polynomial_monomial(Nat.1, b * e) + polynomial_monomial(Nat.2, b * d)
    polynomial_mul(polynomial_constant(c) + polynomial_monomial(Nat.1, b),
        linear_polynomial(d, e)) =
        (polynomial_constant(c * e) + polynomial_monomial(Nat.1, c * d)) +
        (polynomial_monomial(Nat.1, b * e) + polynomial_monomial(Nat.2, b * d))
    polynomial_add_assoc(polynomial_constant(c * e) + polynomial_monomial(Nat.1, c * d),
        polynomial_monomial(Nat.1, b * e), polynomial_monomial(Nat.2, b * d))
    (polynomial_constant(c * e) + polynomial_monomial(Nat.1, c * d)) +
        (polynomial_monomial(Nat.1, b * e) + polynomial_monomial(Nat.2, b * d)) =
        ((polynomial_constant(c * e) + polynomial_monomial(Nat.1, c * d)) +
            polynomial_monomial(Nat.1, b * e)) + polynomial_monomial(Nat.2, b * d)
    polynomial_add_assoc(polynomial_constant(c * e), polynomial_monomial(Nat.1, c * d),
        polynomial_monomial(Nat.1, b * e))
    ((polynomial_constant(c * e) + polynomial_monomial(Nat.1, c * d)) +
        polynomial_monomial(Nat.1, b * e)) + polynomial_monomial(Nat.2, b * d) =
        (polynomial_constant(c * e) + (polynomial_monomial(Nat.1, c * d) +
            polynomial_monomial(Nat.1, b * e))) + polynomial_monomial(Nat.2, b * d)
    polynomial_monomial_add_same(Nat.1, c * d, b * e)
    polynomial_monomial(Nat.1, c * d) + polynomial_monomial(Nat.1, b * e) =
        polynomial_monomial(Nat.1, c * d + b * e)
    polynomial_mul(polynomial_constant(c) + polynomial_monomial(Nat.1, b),
        linear_polynomial(d, e)) =
        polynomial_constant(c * e) + polynomial_monomial(Nat.1, c * d + b * e) +
        polynomial_monomial(Nat.2, b * d)
}

/// The product of a quadratic and a linear polynomial, with the two first
/// monomial pairs merged into single monomials.
lemma quadratic_mul_linear_combined[R: CommRing](a: R, b: R, c: R, d: R, e: R) {
    polynomial_mul(quadratic_polynomial(a, b, c), linear_polynomial(d, e)) =
        polynomial_constant(c * e) + polynomial_monomial(Nat.1, c * d + b * e) +
        polynomial_monomial(Nat.2, b * d + a * e) + polynomial_monomial(Nat.3, a * d)
} by {
    polynomial_mul_add_right(polynomial_constant(c) + polynomial_monomial(Nat.1, b),
        polynomial_monomial(Nat.2, a), linear_polynomial(d, e))
    polynomial_mul(quadratic_polynomial(a, b, c), linear_polynomial(d, e)) =
        polynomial_mul(polynomial_constant(c) + polynomial_monomial(Nat.1, b),
            linear_polynomial(d, e)) +
        polynomial_mul(polynomial_monomial(Nat.2, a), linear_polynomial(d, e))
    quadratic_mul_linear_left_part(b, c, d, e)
    polynomial_mul(polynomial_constant(c) + polynomial_monomial(Nat.1, b),
        linear_polynomial(d, e)) =
        polynomial_constant(c * e) + polynomial_monomial(Nat.1, c * d + b * e) +
        polynomial_monomial(Nat.2, b * d)
    polynomial_mul_add_left(polynomial_monomial(Nat.2, a), polynomial_constant(e),
        polynomial_monomial(Nat.1, d))
    polynomial_mul(polynomial_monomial(Nat.2, a), polynomial_constant(e) +
        polynomial_monomial(Nat.1, d)) =
        polynomial_mul(polynomial_monomial(Nat.2, a), polynomial_constant(e)) +
        polynomial_mul(polynomial_monomial(Nat.2, a), polynomial_monomial(Nat.1, d))
    polynomial_monomial_mul_constant(Nat.2, a, e)
    polynomial_mul(polynomial_monomial(Nat.2, a), polynomial_constant(e)) =
        polynomial_monomial(Nat.2, a * e)
    polynomial_monomial_mul(Nat.2, a, Nat.1, d)
    polynomial_mul(polynomial_monomial(Nat.2, a), polynomial_monomial(Nat.1, d)) =
        polynomial_monomial(Nat.2 + Nat.1, a * d)
    Nat.2 + Nat.1 = Nat.3
    polynomial_mul(polynomial_monomial(Nat.2, a), polynomial_monomial(Nat.1, d)) =
        polynomial_monomial(Nat.3, a * d)
    polynomial_mul(polynomial_monomial(Nat.2, a), linear_polynomial(d, e)) =
        polynomial_monomial(Nat.2, a * e) + polynomial_monomial(Nat.3, a * d)
    polynomial_mul(quadratic_polynomial(a, b, c), linear_polynomial(d, e)) =
        (polynomial_constant(c * e) + polynomial_monomial(Nat.1, c * d + b * e) +
            polynomial_monomial(Nat.2, b * d)) +
        (polynomial_monomial(Nat.2, a * e) + polynomial_monomial(Nat.3, a * d))
    polynomial_add_assoc(polynomial_constant(c * e) + polynomial_monomial(Nat.1, c * d + b * e) +
            polynomial_monomial(Nat.2, b * d), polynomial_monomial(Nat.2, a * e),
        polynomial_monomial(Nat.3, a * d))
    (polynomial_constant(c * e) + polynomial_monomial(Nat.1, c * d + b * e) +
            polynomial_monomial(Nat.2, b * d)) +
        (polynomial_monomial(Nat.2, a * e) + polynomial_monomial(Nat.3, a * d)) =
        ((polynomial_constant(c * e) + polynomial_monomial(Nat.1, c * d + b * e) +
            polynomial_monomial(Nat.2, b * d)) + polynomial_monomial(Nat.2, a * e)) +
        polynomial_monomial(Nat.3, a * d)
    polynomial_add_assoc(polynomial_constant(c * e) + polynomial_monomial(Nat.1, c * d + b * e),
        polynomial_monomial(Nat.2, b * d), polynomial_monomial(Nat.2, a * e))
    ((polynomial_constant(c * e) + polynomial_monomial(Nat.1, c * d + b * e) +
            polynomial_monomial(Nat.2, b * d)) + polynomial_monomial(Nat.2, a * e)) +
        polynomial_monomial(Nat.3, a * d) =
        (polynomial_constant(c * e) + polynomial_monomial(Nat.1, c * d + b * e) +
            (polynomial_monomial(Nat.2, b * d) + polynomial_monomial(Nat.2, a * e))) +
        polynomial_monomial(Nat.3, a * d)
    polynomial_monomial_add_same(Nat.2, b * d, a * e)
    polynomial_monomial(Nat.2, b * d) + polynomial_monomial(Nat.2, a * e) =
        polynomial_monomial(Nat.2, b * d + a * e)
    polynomial_mul(quadratic_polynomial(a, b, c), linear_polynomial(d, e)) =
        polynomial_constant(c * e) + polynomial_monomial(Nat.1, c * d + b * e) +
        polynomial_monomial(Nat.2, b * d + a * e) + polynomial_monomial(Nat.3, a * d)
}

/// The constant coefficient of a cubic polynomial.
theorem cubic_polynomial_coeff_zero[R: Semiring](a: R, b: R, c: R, d: R) {
    cubic_polynomial(a, b, c, d).coeff(Nat.0) = d
} by {
    polynomial_add_coeff(polynomial_constant(d) + polynomial_monomial(Nat.1, c) +
        polynomial_monomial(Nat.2, b), polynomial_monomial(Nat.3, a), Nat.0)
    cubic_polynomial(a, b, c, d).coeff(Nat.0) =
        (polynomial_constant(d) + polynomial_monomial(Nat.1, c) +
            polynomial_monomial(Nat.2, b)).coeff(Nat.0) +
        polynomial_monomial(Nat.3, a).coeff(Nat.0)
    polynomial_add_coeff(polynomial_constant(d) + polynomial_monomial(Nat.1, c),
        polynomial_monomial(Nat.2, b), Nat.0)
    (polynomial_constant(d) + polynomial_monomial(Nat.1, c) +
        polynomial_monomial(Nat.2, b)).coeff(Nat.0) =
        (polynomial_constant(d) + polynomial_monomial(Nat.1, c)).coeff(Nat.0) +
        polynomial_monomial(Nat.2, b).coeff(Nat.0)
    polynomial_add_coeff(polynomial_constant(d), polynomial_monomial(Nat.1, c), Nat.0)
    (polynomial_constant(d) + polynomial_monomial(Nat.1, c)).coeff(Nat.0) =
        polynomial_constant(d).coeff(Nat.0) + polynomial_monomial(Nat.1, c).coeff(Nat.0)
    polynomial_constant_coeff_zero(d)
    polynomial_constant(d).coeff(Nat.0) = d
    Nat.0 != Nat.1
    polynomial_monomial_coeff_of_ne(Nat.1, c, Nat.0)
    polynomial_monomial(Nat.1, c).coeff(Nat.0) = R.0
    (polynomial_constant(d) + polynomial_monomial(Nat.1, c)).coeff(Nat.0) = d + R.0
    d + R.0 = d
    Nat.0 != Nat.2
    polynomial_monomial_coeff_of_ne(Nat.2, b, Nat.0)
    polynomial_monomial(Nat.2, b).coeff(Nat.0) = R.0
    (polynomial_constant(d) + polynomial_monomial(Nat.1, c) +
        polynomial_monomial(Nat.2, b)).coeff(Nat.0) = d + R.0
    d + R.0 = d
    Nat.0 != Nat.3
    polynomial_monomial_coeff_of_ne(Nat.3, a, Nat.0)
    polynomial_monomial(Nat.3, a).coeff(Nat.0) = R.0
    cubic_polynomial(a, b, c, d).coeff(Nat.0) = d + R.0
    d + R.0 = d
    cubic_polynomial(a, b, c, d).coeff(Nat.0) = d
}

/// The linear coefficient of a cubic polynomial.
theorem cubic_polynomial_coeff_one[R: Semiring](a: R, b: R, c: R, d: R) {
    cubic_polynomial(a, b, c, d).coeff(Nat.1) = c
} by {
    polynomial_add_coeff(polynomial_constant(d) + polynomial_monomial(Nat.1, c) +
        polynomial_monomial(Nat.2, b), polynomial_monomial(Nat.3, a), Nat.1)
    cubic_polynomial(a, b, c, d).coeff(Nat.1) =
        (polynomial_constant(d) + polynomial_monomial(Nat.1, c) +
            polynomial_monomial(Nat.2, b)).coeff(Nat.1) +
        polynomial_monomial(Nat.3, a).coeff(Nat.1)
    polynomial_add_coeff(polynomial_constant(d) + polynomial_monomial(Nat.1, c),
        polynomial_monomial(Nat.2, b), Nat.1)
    (polynomial_constant(d) + polynomial_monomial(Nat.1, c) +
        polynomial_monomial(Nat.2, b)).coeff(Nat.1) =
        (polynomial_constant(d) + polynomial_monomial(Nat.1, c)).coeff(Nat.1) +
        polynomial_monomial(Nat.2, b).coeff(Nat.1)
    polynomial_add_coeff(polynomial_constant(d), polynomial_monomial(Nat.1, c), Nat.1)
    (polynomial_constant(d) + polynomial_monomial(Nat.1, c)).coeff(Nat.1) =
        polynomial_constant(d).coeff(Nat.1) + polynomial_monomial(Nat.1, c).coeff(Nat.1)
    Nat.1 != Nat.0
    polynomial_constant_coeff_of_ne_zero(d, Nat.1)
    polynomial_constant(d).coeff(Nat.1) = R.0
    polynomial_monomial_coeff_self(Nat.1, c)
    polynomial_monomial(Nat.1, c).coeff(Nat.1) = c
    (polynomial_constant(d) + polynomial_monomial(Nat.1, c)).coeff(Nat.1) = R.0 + c
    R.0 + c = c
    Nat.1 != Nat.2
    polynomial_monomial_coeff_of_ne(Nat.2, b, Nat.1)
    polynomial_monomial(Nat.2, b).coeff(Nat.1) = R.0
    (polynomial_constant(d) + polynomial_monomial(Nat.1, c) +
        polynomial_monomial(Nat.2, b)).coeff(Nat.1) = c + R.0
    c + R.0 = c
    lt_suc(Nat.1)
    Nat.1 < Nat.2
    lt_suc(Nat.2)
    Nat.2 < Nat.3
    lt_trans[Nat](Nat.1, Nat.2, Nat.3)
    Nat.1 < Nat.3
    lt_imp_ne[Nat](Nat.1, Nat.3)
    Nat.1 != Nat.3
    polynomial_monomial_coeff_of_ne(Nat.3, a, Nat.1)
    polynomial_monomial(Nat.3, a).coeff(Nat.1) = R.0
    cubic_polynomial(a, b, c, d).coeff(Nat.1) = c + R.0
    c + R.0 = c
    cubic_polynomial(a, b, c, d).coeff(Nat.1) = c
}

/// The quadratic coefficient of a cubic polynomial.
theorem cubic_polynomial_coeff_two[R: Semiring](a: R, b: R, c: R, d: R) {
    cubic_polynomial(a, b, c, d).coeff(Nat.2) = b
} by {
    polynomial_add_coeff(polynomial_constant(d) + polynomial_monomial(Nat.1, c) +
        polynomial_monomial(Nat.2, b), polynomial_monomial(Nat.3, a), Nat.2)
    cubic_polynomial(a, b, c, d).coeff(Nat.2) =
        (polynomial_constant(d) + polynomial_monomial(Nat.1, c) +
            polynomial_monomial(Nat.2, b)).coeff(Nat.2) +
        polynomial_monomial(Nat.3, a).coeff(Nat.2)
    polynomial_add_coeff(polynomial_constant(d) + polynomial_monomial(Nat.1, c),
        polynomial_monomial(Nat.2, b), Nat.2)
    (polynomial_constant(d) + polynomial_monomial(Nat.1, c) +
        polynomial_monomial(Nat.2, b)).coeff(Nat.2) =
        (polynomial_constant(d) + polynomial_monomial(Nat.1, c)).coeff(Nat.2) +
        polynomial_monomial(Nat.2, b).coeff(Nat.2)
    polynomial_add_coeff(polynomial_constant(d), polynomial_monomial(Nat.1, c), Nat.2)
    (polynomial_constant(d) + polynomial_monomial(Nat.1, c)).coeff(Nat.2) =
        polynomial_constant(d).coeff(Nat.2) + polynomial_monomial(Nat.1, c).coeff(Nat.2)
    Nat.2 != Nat.0
    polynomial_constant_coeff_of_ne_zero(d, Nat.2)
    polynomial_constant(d).coeff(Nat.2) = R.0
    Nat.1 != Nat.2
    polynomial_monomial_coeff_of_ne(Nat.1, c, Nat.2)
    polynomial_monomial(Nat.1, c).coeff(Nat.2) = R.0
    (polynomial_constant(d) + polynomial_monomial(Nat.1, c)).coeff(Nat.2) = R.0 + R.0
    R.0 + R.0 = R.0
    polynomial_monomial_coeff_self(Nat.2, b)
    polynomial_monomial(Nat.2, b).coeff(Nat.2) = b
    (polynomial_constant(d) + polynomial_monomial(Nat.1, c) +
        polynomial_monomial(Nat.2, b)).coeff(Nat.2) = R.0 + b
    R.0 + b = b
    Nat.2 != Nat.3
    polynomial_monomial_coeff_of_ne(Nat.3, a, Nat.2)
    polynomial_monomial(Nat.3, a).coeff(Nat.2) = R.0
    cubic_polynomial(a, b, c, d).coeff(Nat.2) = b + R.0
    b + R.0 = b
    cubic_polynomial(a, b, c, d).coeff(Nat.2) = b
}

/// The cubic coefficient of a cubic polynomial.
theorem cubic_polynomial_coeff_three[R: Semiring](a: R, b: R, c: R, d: R) {
    cubic_polynomial(a, b, c, d).coeff(Nat.3) = a
} by {
    polynomial_add_coeff(polynomial_constant(d) + polynomial_monomial(Nat.1, c) +
        polynomial_monomial(Nat.2, b), polynomial_monomial(Nat.3, a), Nat.3)
    cubic_polynomial(a, b, c, d).coeff(Nat.3) =
        (polynomial_constant(d) + polynomial_monomial(Nat.1, c) +
            polynomial_monomial(Nat.2, b)).coeff(Nat.3) +
        polynomial_monomial(Nat.3, a).coeff(Nat.3)
    polynomial_add_coeff(polynomial_constant(d) + polynomial_monomial(Nat.1, c),
        polynomial_monomial(Nat.2, b), Nat.3)
    (polynomial_constant(d) + polynomial_monomial(Nat.1, c) +
        polynomial_monomial(Nat.2, b)).coeff(Nat.3) =
        (polynomial_constant(d) + polynomial_monomial(Nat.1, c)).coeff(Nat.3) +
        polynomial_monomial(Nat.2, b).coeff(Nat.3)
    polynomial_add_coeff(polynomial_constant(d), polynomial_monomial(Nat.1, c), Nat.3)
    (polynomial_constant(d) + polynomial_monomial(Nat.1, c)).coeff(Nat.3) =
        polynomial_constant(d).coeff(Nat.3) + polynomial_monomial(Nat.1, c).coeff(Nat.3)
    Nat.3 != Nat.0
    polynomial_constant_coeff_of_ne_zero(d, Nat.3)
    polynomial_constant(d).coeff(Nat.3) = R.0
    lt_suc(Nat.1)
    Nat.1 < Nat.2
    lt_suc(Nat.2)
    Nat.2 < Nat.3
    lt_trans[Nat](Nat.1, Nat.2, Nat.3)
    Nat.1 < Nat.3
    lt_imp_ne[Nat](Nat.1, Nat.3)
    Nat.1 != Nat.3
    polynomial_monomial_coeff_of_ne(Nat.1, c, Nat.3)
    polynomial_monomial(Nat.1, c).coeff(Nat.3) = R.0
    (polynomial_constant(d) + polynomial_monomial(Nat.1, c)).coeff(Nat.3) = R.0 + R.0
    R.0 + R.0 = R.0
    Nat.2 != Nat.3
    polynomial_monomial_coeff_of_ne(Nat.2, b, Nat.3)
    polynomial_monomial(Nat.2, b).coeff(Nat.3) = R.0
    (polynomial_constant(d) + polynomial_monomial(Nat.1, c) +
        polynomial_monomial(Nat.2, b)).coeff(Nat.3) = R.0 + R.0
    R.0 + R.0 = R.0
    polynomial_monomial_coeff_self(Nat.3, a)
    polynomial_monomial(Nat.3, a).coeff(Nat.3) = a
    cubic_polynomial(a, b, c, d).coeff(Nat.3) = R.0 + a
    R.0 + a = a
    cubic_polynomial(a, b, c, d).coeff(Nat.3) = a
}

/// The value of a cubic polynomial at a point.
theorem cubic_polynomial_eval[R: CommRing](a: R, b: R, c: R, d: R, x: R) {
    polynomial_eval(cubic_polynomial(a, b, c, d), x) = d + c * x + b * x * x + a * x * x * x
} by {
    polynomial_eval_add(polynomial_constant(d) + polynomial_monomial(Nat.1, c) +
        polynomial_monomial(Nat.2, b), polynomial_monomial(Nat.3, a), x)
    polynomial_eval(cubic_polynomial(a, b, c, d), x) =
        polynomial_eval(polynomial_constant(d) + polynomial_monomial(Nat.1, c) +
            polynomial_monomial(Nat.2, b), x) +
        polynomial_eval(polynomial_monomial(Nat.3, a), x)
    polynomial_eval_add(polynomial_constant(d) + polynomial_monomial(Nat.1, c),
        polynomial_monomial(Nat.2, b), x)
    polynomial_eval(polynomial_constant(d) + polynomial_monomial(Nat.1, c) +
        polynomial_monomial(Nat.2, b), x) =
        polynomial_eval(polynomial_constant(d) + polynomial_monomial(Nat.1, c), x) +
        polynomial_eval(polynomial_monomial(Nat.2, b), x)
    polynomial_eval_add(polynomial_constant(d), polynomial_monomial(Nat.1, c), x)
    polynomial_eval(polynomial_constant(d) + polynomial_monomial(Nat.1, c), x) =
        polynomial_eval(polynomial_constant(d), x) + polynomial_eval(polynomial_monomial(Nat.1, c), x)
    polynomial_eval_constant(d, x)
    polynomial_eval(polynomial_constant(d), x) = d
    polynomial_eval_monomial(Nat.1, c, x)
    polynomial_eval(polynomial_monomial(Nat.1, c), x) = c * x.pow(Nat.1)
    pow_one(x)
    x.pow(Nat.1) = x
    polynomial_eval(polynomial_monomial(Nat.1, c), x) = c * x
    polynomial_eval(polynomial_constant(d) + polynomial_monomial(Nat.1, c), x) = d + c * x
    polynomial_eval_monomial(Nat.2, b, x)
    polynomial_eval(polynomial_monomial(Nat.2, b), x) = b * x.pow(Nat.2)
    cubic_pow_two(x)
    x.pow(Nat.2) = x * x
    polynomial_eval(polynomial_monomial(Nat.2, b), x) = b * (x * x)
    polynomial_eval(polynomial_constant(d) + polynomial_monomial(Nat.1, c) +
        polynomial_monomial(Nat.2, b), x) = d + c * x + b * (x * x)
    b * (x * x) = b * x * x
    polynomial_eval(polynomial_constant(d) + polynomial_monomial(Nat.1, c) +
        polynomial_monomial(Nat.2, b), x) = d + c * x + b * x * x
    polynomial_eval_monomial(Nat.3, a, x)
    polynomial_eval(polynomial_monomial(Nat.3, a), x) = a * x.pow(Nat.3)
    cubic_pow_three(x)
    x.pow(Nat.3) = x * x * x
    polynomial_eval(polynomial_monomial(Nat.3, a), x) = a * (x * x * x)
    polynomial_eval(cubic_polynomial(a, b, c, d), x) = d + c * x + b * x * x + a * (x * x * x)
    a * (x * x * x) = a * x * x * x
    polynomial_eval(cubic_polynomial(a, b, c, d), x) = d + c * x + b * x * x + a * x * x * x
}

/// The product of the three monic linear factors `X - r1`, `X - r2`, `X - r3`
/// is the monic cubic with Vieta coefficients.
theorem cubic_vieta_factorization[R: CommRing](r1: R, r2: R, r3: R) {
    polynomial_mul(polynomial_mul(linear_polynomial(R.1, -r1),
        linear_polynomial(R.1, -r2)), linear_polynomial(R.1, -r3)) =
        cubic_polynomial(R.1, -(r1 + r2 + r3), r1 * r2 + r1 * r3 + r2 * r3, -(r1 * r2 * r3))
} by {
    polynomial_linear_mul_quadratic(r1, r2)
    polynomial_mul(linear_polynomial(R.1, -r1), linear_polynomial(R.1, -r2)) =
        quadratic_polynomial(R.1, -(r1 + r2), r1 * r2)
    polynomial_mul(polynomial_mul(linear_polynomial(R.1, -r1),
        linear_polynomial(R.1, -r2)), linear_polynomial(R.1, -r3)) =
        polynomial_mul(quadratic_polynomial(R.1, -(r1 + r2), r1 * r2),
            linear_polynomial(R.1, -r3))
    quadratic_mul_linear_combined(R.1, -(r1 + r2), r1 * r2, R.1, -r3)
    polynomial_mul(quadratic_polynomial(R.1, -(r1 + r2), r1 * r2),
        linear_polynomial(R.1, -r3)) =
        polynomial_constant((r1 * r2) * -r3) +
        polynomial_monomial(Nat.1, (r1 * r2) * R.1 + (-(r1 + r2)) * -r3) +
        polynomial_monomial(Nat.2, (-(r1 + r2)) * R.1 + R.1 * -r3) +
        polynomial_monomial(Nat.3, R.1 * R.1)
    mul_neg_right(r1 * r2, r3)
    (r1 * r2) * -r3 = -((r1 * r2) * r3)
    (r1 * r2) * r3 = r1 * r2 * r3
    (r1 * r2) * -r3 = -(r1 * r2 * r3)
    polynomial_constant((r1 * r2) * -r3) = polynomial_constant(-(r1 * r2 * r3))
    (r1 * r2) * R.1 = r1 * r2
    mul_neg_left(r1 + r2, -r3)
    (-(r1 + r2)) * -r3 = -((r1 + r2) * -r3)
    mul_neg_right(r1 + r2, r3)
    (r1 + r2) * -r3 = -((r1 + r2) * r3)
    (-(r1 + r2)) * -r3 = -(-((r1 + r2) * r3))
    (r1 + r2) * r3 = r1 * r3 + r2 * r3
    (-(r1 + r2)) * -r3 = (r1 + r2) * r3
    (-(r1 + r2)) * -r3 = r1 * r3 + r2 * r3
    (r1 * r2) * R.1 + (-(r1 + r2)) * -r3 = r1 * r2 + r1 * r3 + r2 * r3
    polynomial_monomial(Nat.1, (r1 * r2) * R.1 + (-(r1 + r2)) * -r3) =
        polynomial_monomial(Nat.1, r1 * r2 + r1 * r3 + r2 * r3)
    (-(r1 + r2)) * R.1 = -(r1 + r2)
    R.1 * -r3 = -r3
    (-(r1 + r2)) * R.1 + R.1 * -r3 = -(r1 + r2) + -r3
    inverse_add(r1 + r2, r3)
    -((r1 + r2) + r3) = -r3 + -(r1 + r2)
    -(r1 + r2) + -r3 = -r3 + -(r1 + r2)
    (-(r1 + r2)) * R.1 + R.1 * -r3 = -((r1 + r2) + r3)
    (-(r1 + r2)) * R.1 + R.1 * -r3 = -(r1 + r2 + r3)
    polynomial_monomial(Nat.2, (-(r1 + r2)) * R.1 + R.1 * -r3) =
        polynomial_monomial(Nat.2, -(r1 + r2 + r3))
    R.1 * R.1 = R.1
    polynomial_monomial(Nat.3, R.1 * R.1) = polynomial_monomial(Nat.3, R.1)
    polynomial_mul(polynomial_mul(linear_polynomial(R.1, -r1),
        linear_polynomial(R.1, -r2)), linear_polynomial(R.1, -r3)) =
        polynomial_constant(-(r1 * r2 * r3)) +
        polynomial_monomial(Nat.1, r1 * r2 + r1 * r3 + r2 * r3) +
        polynomial_monomial(Nat.2, -(r1 + r2 + r3)) +
        polynomial_monomial(Nat.3, R.1)
    cubic_polynomial(R.1, -(r1 + r2 + r3), r1 * r2 + r1 * r3 + r2 * r3, -(r1 * r2 * r3)) =
        polynomial_constant(-(r1 * r2 * r3)) +
        polynomial_monomial(Nat.1, r1 * r2 + r1 * r3 + r2 * r3) +
        polynomial_monomial(Nat.2, -(r1 + r2 + r3)) + polynomial_monomial(Nat.3, R.1)
    polynomial_mul(polynomial_mul(linear_polynomial(R.1, -r1),
        linear_polynomial(R.1, -r2)), linear_polynomial(R.1, -r3)) =
        cubic_polynomial(R.1, -(r1 + r2 + r3), r1 * r2 + r1 * r3 + r2 * r3, -(r1 * r2 * r3))
}

/// The value of the monic cubic with roots `r1`, `r2`, `r3` at `r1` is zero.
theorem cubic_root_vieta_a[R: CommRing](r1: R, r2: R, r3: R) {
    polynomial_eval(cubic_polynomial(R.1, -(r1 + r2 + r3), r1 * r2 + r1 * r3 + r2 * r3, -(r1 * r2 * r3)), r1) = R.0
} by {
    cubic_vieta_factorization(r1, r2, r3)
    polynomial_mul(polynomial_mul(linear_polynomial(R.1, -r1),
        linear_polynomial(R.1, -r2)), linear_polynomial(R.1, -r3)) =
        cubic_polynomial(R.1, -(r1 + r2 + r3), r1 * r2 + r1 * r3 + r2 * r3, -(r1 * r2 * r3))
    polynomial_eval(cubic_polynomial(R.1, -(r1 + r2 + r3), r1 * r2 + r1 * r3 + r2 * r3, -(r1 * r2 * r3)), r1) =
        polynomial_eval(polynomial_mul(polynomial_mul(linear_polynomial(R.1, -r1),
            linear_polynomial(R.1, -r2)), linear_polynomial(R.1, -r3)), r1)
    polynomial_eval_mul(polynomial_mul(linear_polynomial(R.1, -r1),
        linear_polynomial(R.1, -r2)), linear_polynomial(R.1, -r3), r1)
    polynomial_eval(polynomial_mul(polynomial_mul(linear_polynomial(R.1, -r1),
        linear_polynomial(R.1, -r2)), linear_polynomial(R.1, -r3)), r1) =
        polynomial_eval(polynomial_mul(linear_polynomial(R.1, -r1), linear_polynomial(R.1, -r2)), r1) * polynomial_eval(linear_polynomial(R.1, -r3), r1)
    polynomial_eval_mul(linear_polynomial(R.1, -r1), linear_polynomial(R.1, -r2), r1)
    polynomial_eval(polynomial_mul(linear_polynomial(R.1, -r1),
        linear_polynomial(R.1, -r2)), r1) =
        polynomial_eval(linear_polynomial(R.1, -r1), r1) * polynomial_eval(linear_polynomial(R.1, -r2), r1)
    linear_polynomial_neg_root(r1)
    polynomial_eval(linear_polynomial(R.1, -r1), r1) = R.0
    polynomial_eval(polynomial_mul(linear_polynomial(R.1, -r1),
        linear_polynomial(R.1, -r2)), r1) = R.0 * polynomial_eval(linear_polynomial(R.1, -r2), r1)
    R.0 * polynomial_eval(linear_polynomial(R.1, -r2), r1) = R.0
    polynomial_eval(polynomial_mul(linear_polynomial(R.1, -r1),
        linear_polynomial(R.1, -r2)), r1) = R.0
    polynomial_eval(polynomial_mul(polynomial_mul(linear_polynomial(R.1, -r1),
        linear_polynomial(R.1, -r2)), linear_polynomial(R.1, -r3)), r1) =
        R.0 * polynomial_eval(linear_polynomial(R.1, -r3), r1)
    R.0 * polynomial_eval(linear_polynomial(R.1, -r3), r1) = R.0
    polynomial_eval(cubic_polynomial(R.1, -(r1 + r2 + r3), r1 * r2 + r1 * r3 + r2 * r3, -(r1 * r2 * r3)), r1) = R.0
}

/// The value of the monic cubic with roots `r1`, `r2`, `r3` at `r2` is zero.
theorem cubic_root_vieta_b[R: CommRing](r1: R, r2: R, r3: R) {
    polynomial_eval(cubic_polynomial(R.1, -(r1 + r2 + r3), r1 * r2 + r1 * r3 + r2 * r3, -(r1 * r2 * r3)), r2) = R.0
} by {
    cubic_vieta_factorization(r1, r2, r3)
    polynomial_mul(polynomial_mul(linear_polynomial(R.1, -r1),
        linear_polynomial(R.1, -r2)), linear_polynomial(R.1, -r3)) =
        cubic_polynomial(R.1, -(r1 + r2 + r3), r1 * r2 + r1 * r3 + r2 * r3, -(r1 * r2 * r3))
    polynomial_eval(cubic_polynomial(R.1, -(r1 + r2 + r3), r1 * r2 + r1 * r3 + r2 * r3, -(r1 * r2 * r3)), r2) =
        polynomial_eval(polynomial_mul(polynomial_mul(linear_polynomial(R.1, -r1),
            linear_polynomial(R.1, -r2)), linear_polynomial(R.1, -r3)), r2)
    polynomial_eval_mul(polynomial_mul(linear_polynomial(R.1, -r1),
        linear_polynomial(R.1, -r2)), linear_polynomial(R.1, -r3), r2)
    polynomial_eval(polynomial_mul(polynomial_mul(linear_polynomial(R.1, -r1),
        linear_polynomial(R.1, -r2)), linear_polynomial(R.1, -r3)), r2) =
        polynomial_eval(polynomial_mul(linear_polynomial(R.1, -r1), linear_polynomial(R.1, -r2)), r2) * polynomial_eval(linear_polynomial(R.1, -r3), r2)
    polynomial_eval_mul(linear_polynomial(R.1, -r1), linear_polynomial(R.1, -r2), r2)
    polynomial_eval(polynomial_mul(linear_polynomial(R.1, -r1),
        linear_polynomial(R.1, -r2)), r2) =
        polynomial_eval(linear_polynomial(R.1, -r1), r2) * polynomial_eval(linear_polynomial(R.1, -r2), r2)
    linear_polynomial_neg_root(r2)
    polynomial_eval(linear_polynomial(R.1, -r2), r2) = R.0
    polynomial_eval(polynomial_mul(linear_polynomial(R.1, -r1),
        linear_polynomial(R.1, -r2)), r2) =
        polynomial_eval(linear_polynomial(R.1, -r1), r2) * R.0
    polynomial_eval(linear_polynomial(R.1, -r1), r2) * R.0 = R.0
    polynomial_eval(polynomial_mul(linear_polynomial(R.1, -r1),
        linear_polynomial(R.1, -r2)), r2) = R.0
    polynomial_eval(polynomial_mul(polynomial_mul(linear_polynomial(R.1, -r1),
        linear_polynomial(R.1, -r2)), linear_polynomial(R.1, -r3)), r2) =
        R.0 * polynomial_eval(linear_polynomial(R.1, -r3), r2)
    R.0 * polynomial_eval(linear_polynomial(R.1, -r3), r2) = R.0
    polynomial_eval(cubic_polynomial(R.1, -(r1 + r2 + r3), r1 * r2 + r1 * r3 + r2 * r3, -(r1 * r2 * r3)), r2) = R.0
}

/// The value of the monic cubic with roots `r1`, `r2`, `r3` at `r3` is zero.
theorem cubic_root_vieta_c[R: CommRing](r1: R, r2: R, r3: R) {
    polynomial_eval(cubic_polynomial(R.1, -(r1 + r2 + r3), r1 * r2 + r1 * r3 + r2 * r3, -(r1 * r2 * r3)), r3) = R.0
} by {
    cubic_vieta_factorization(r1, r2, r3)
    polynomial_mul(polynomial_mul(linear_polynomial(R.1, -r1),
        linear_polynomial(R.1, -r2)), linear_polynomial(R.1, -r3)) =
        cubic_polynomial(R.1, -(r1 + r2 + r3), r1 * r2 + r1 * r3 + r2 * r3, -(r1 * r2 * r3))
    polynomial_eval(cubic_polynomial(R.1, -(r1 + r2 + r3), r1 * r2 + r1 * r3 + r2 * r3, -(r1 * r2 * r3)), r3) =
        polynomial_eval(polynomial_mul(polynomial_mul(linear_polynomial(R.1, -r1),
            linear_polynomial(R.1, -r2)), linear_polynomial(R.1, -r3)), r3)
    polynomial_eval_mul(polynomial_mul(linear_polynomial(R.1, -r1),
        linear_polynomial(R.1, -r2)), linear_polynomial(R.1, -r3), r3)
    polynomial_eval(polynomial_mul(polynomial_mul(linear_polynomial(R.1, -r1),
        linear_polynomial(R.1, -r2)), linear_polynomial(R.1, -r3)), r3) =
        polynomial_eval(polynomial_mul(linear_polynomial(R.1, -r1), linear_polynomial(R.1, -r2)), r3) * polynomial_eval(linear_polynomial(R.1, -r3), r3)
    polynomial_eval_mul(linear_polynomial(R.1, -r1), linear_polynomial(R.1, -r2), r3)
    polynomial_eval(polynomial_mul(linear_polynomial(R.1, -r1),
        linear_polynomial(R.1, -r2)), r3) =
        polynomial_eval(linear_polynomial(R.1, -r1), r3) * polynomial_eval(linear_polynomial(R.1, -r2), r3)
    linear_polynomial_neg_root(r3)
    polynomial_eval(linear_polynomial(R.1, -r3), r3) = R.0
    polynomial_eval(polynomial_mul(polynomial_mul(linear_polynomial(R.1, -r1),
        linear_polynomial(R.1, -r2)), linear_polynomial(R.1, -r3)), r3) =
        polynomial_eval(polynomial_mul(linear_polynomial(R.1, -r1),
            linear_polynomial(R.1, -r2)), r3) * R.0
    polynomial_eval(polynomial_mul(linear_polynomial(R.1, -r1),
        linear_polynomial(R.1, -r2)), r3) * R.0 = R.0
    polynomial_eval(cubic_polynomial(R.1, -(r1 + r2 + r3), r1 * r2 + r1 * r3 + r2 * r3, -(r1 * r2 * r3)), r3) = R.0
}

/// The simple Vieta relations: the monic cubic with roots `r1`, `r2`, `r3`
/// has quadratic coefficient `-(r1 + r2 + r3)`.
theorem vieta_cubic_coeff_quadratic[R: CommRing](r1: R, r2: R, r3: R) {
    cubic_polynomial(R.1, -(r1 + r2 + r3), r1 * r2 + r1 * r3 + r2 * r3, -(r1 * r2 * r3)).coeff(Nat.2) = -(r1 + r2 + r3)
} by {
    cubic_polynomial_coeff_two(R.1, -(r1 + r2 + r3), r1 * r2 + r1 * r3 + r2 * r3, -(r1 * r2 * r3))
}

/// The simple Vieta relations: the monic cubic with roots `r1`, `r2`, `r3`
/// has linear coefficient `r1 * r2 + r1 * r3 + r2 * r3`.
theorem vieta_cubic_coeff_linear[R: CommRing](r1: R, r2: R, r3: R) {
    cubic_polynomial(R.1, -(r1 + r2 + r3), r1 * r2 + r1 * r3 + r2 * r3, -(r1 * r2 * r3)).coeff(Nat.1) = r1 * r2 + r1 * r3 + r2 * r3
} by {
    cubic_polynomial_coeff_one(R.1, -(r1 + r2 + r3), r1 * r2 + r1 * r3 + r2 * r3, -(r1 * r2 * r3))
}

/// The simple Vieta relations: the monic cubic with roots `r1`, `r2`, `r3`
/// has constant coefficient `-(r1 * r2 * r3)`.
theorem vieta_cubic_coeff_constant[R: CommRing](r1: R, r2: R, r3: R) {
    cubic_polynomial(R.1, -(r1 + r2 + r3), r1 * r2 + r1 * r3 + r2 * r3, -(r1 * r2 * r3)).coeff(Nat.0) = -(r1 * r2 * r3)
} by {
    cubic_polynomial_coeff_zero(R.1, -(r1 + r2 + r3), r1 * r2 + r1 * r3 + r2 * r3, -(r1 * r2 * r3))
}

/// The simple Vieta relations: the monic cubic with roots `r1`, `r2`, `r3`
/// has leading coefficient one.
theorem vieta_cubic_coeff_leading[R: CommRing](r1: R, r2: R, r3: R) {
    cubic_polynomial(R.1, -(r1 + r2 + r3), r1 * r2 + r1 * r3 + r2 * r3, -(r1 * r2 * r3)).coeff(Nat.3) = R.1
} by {
    cubic_polynomial_coeff_three(R.1, -(r1 + r2 + r3), r1 * r2 + r1 * r3 + r2 * r3, -(r1 * r2 * r3))
}
