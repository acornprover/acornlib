/// Monomial structure: products of monomials are monomials, constant
/// polynomials are the monomials of exponent zero, and the vanishing of a
/// monomial is the vanishing of its coefficient.

from nat import Nat, add_sub, add_cancels_right, lt_suc, lte_and_lt
from semiring import Semiring
from comm_ring import CommRing
from algebra.comm_semigroup import CommSemigroup
from polynomial import Polynomial, polynomial_monomial, polynomial_constant,
    polynomial_monomial_coeff, polynomial_monomial_coeff_self, polynomial_monomial_coeff_of_ne,
    polynomial_zero_coeff, polynomial_ext_pointwise, polynomial_eq_coeff_at,
    polynomial_mul, polynomial_mul_coeff, polynomial_mul_coeff_apply, polynomial_mul_term_coeff,
    polynomial_eval
from polynomial.polynomial_eval_deep import polynomial_eval_monomial
from data.nat.nat_range_sum import range_sum, range_sum_congr
from data.nat.nat_residue_range_sum import point_fn, point_fn_at, point_fn_off, range_sum_point_fn,
    range_sum_zero_of_vanishes
from polynomial_mul_comm import mul_coeff_eq_range_sum

numerals Nat

/// Every natural number is at most its own sum with another.
lemma monomial_nat_lte_add_left(m: Nat, n: Nat) {
    m <= m + n
} by {
    m + n = m + n
}

/// The attribute monomial is the same polynomial as the named monomial.
lemma polynomial_monomial_alias[R: Semiring](n: Nat, r: R) {
    Polynomial[R].monomial(n, r) = polynomial_monomial(n, r)
} by {
    Polynomial[R].monomial(n, r) = polynomial_monomial(n, r)
}

/// A monomial has its coefficient at its exponent.
lemma monomial_coeff_self[R: Semiring](n: Nat, r: R) {
    polynomial_monomial(n, r).coeff(n) = r
} by {
    polynomial_monomial_alias(n, r)
    Polynomial[R].monomial(n, r) = polynomial_monomial(n, r)
    polynomial_monomial_coeff_self(n, r)
    Polynomial[R].monomial(n, r).coeff(n) = r
    polynomial_monomial(n, r).coeff(n) = r
}

/// A monomial vanishes away from its exponent.
lemma monomial_coeff_of_ne[R: Semiring](n: Nat, r: R, k: Nat) {
    k != n implies polynomial_monomial(n, r).coeff(k) = R.0
} by {
    if k != n {
        polynomial_monomial_alias(n, r)
        Polynomial[R].monomial(n, r) = polynomial_monomial(n, r)
        polynomial_monomial_coeff_of_ne[R](n, r, k)
        Polynomial[R].monomial(n, r).coeff(k) = R.0
        polynomial_monomial(n, r).coeff(k) = R.0
    }
}

/// The constant polynomial with coefficient `r` is the monomial of exponent
/// zero.
theorem polynomial_constant_eq_monomial_zero[R: Semiring](r: R) {
    polynomial_constant(r) = polynomial_monomial(Nat.0, r)
} by {
    polynomial_constant(r) = polynomial_monomial(Nat.0, r)
}

/// Away from the sum exponent, one convolution summand of a monomial product
/// vanishes.
lemma monomial_mul_coeff_vanishes_point[R: CommRing](
    m: Nat, a: R, n: Nat, b: R, k: Nat, i: Nat
) {
    k != m + n and i < k.suc implies
    polynomial_mul_term_coeff(polynomial_monomial(m, a), polynomial_monomial(n, b), k, i) =
        R.0
} by {
    if k != m + n and i < k.suc {
        if i = m {
            i = m
            i <= k
            m <= k
            add_sub(k, m)
            k - m + m = k
            if k - m = n {
                k - m + m = n + m
                k = n + m
                n + m = m + n
                k = m + n
                false
            }
            k - m != n
            Polynomial[R].new(polynomial_monomial_coeff[R](n, b)) =
                Option.some(polynomial_monomial(n, b))
            polynomial_monomial(n, b).coeff = polynomial_monomial_coeff[R](n, b)
            polynomial_monomial(n, b).coeff(k - m) = polynomial_monomial_coeff[R](n, b, k - m)
            polynomial_monomial_coeff[R](n, b, k - m) = R.0
            polynomial_monomial(n, b).coeff(k - m) = R.0
            polynomial_mul_term_coeff(polynomial_monomial(m, a),
                polynomial_monomial(n, b), k, i) =
                polynomial_monomial(m, a).coeff(i) *
                polynomial_monomial(n, b).coeff(k - i)
            k - i = k - m
            polynomial_mul_term_coeff(polynomial_monomial(m, a),
                polynomial_monomial(n, b), k, i) =
                polynomial_monomial(m, a).coeff(i) * R.0
            polynomial_monomial(m, a).coeff(i) * R.0 = R.0
            polynomial_mul_term_coeff(polynomial_monomial(m, a),
                polynomial_monomial(n, b), k, i) = R.0
        } else {
            i != m
            Polynomial[R].new(polynomial_monomial_coeff[R](m, a)) =
                Option.some(polynomial_monomial(m, a))
            polynomial_monomial(m, a).coeff = polynomial_monomial_coeff[R](m, a)
            polynomial_monomial(m, a).coeff(i) = polynomial_monomial_coeff[R](m, a, i)
            polynomial_monomial_coeff[R](m, a, i) = R.0
            polynomial_monomial(m, a).coeff(i) = R.0
            polynomial_mul_term_coeff(polynomial_monomial(m, a),
                polynomial_monomial(n, b), k, i) =
                polynomial_monomial(m, a).coeff(i) *
                polynomial_monomial(n, b).coeff(k - i)
            polynomial_mul_term_coeff(polynomial_monomial(m, a),
                polynomial_monomial(n, b), k, i) = R.0 * polynomial_monomial(n, b).coeff(k - i)
            R.0 * polynomial_monomial(n, b).coeff(k - i) = R.0
            polynomial_mul_term_coeff(polynomial_monomial(m, a),
                polynomial_monomial(n, b), k, i) = R.0
        }
    }
}

/// A product of two monomials is the monomial of the sum of exponents.
theorem polynomial_monomial_mul[R: CommRing](m: Nat, a: R, n: Nat, b: R) {
    polynomial_mul(polynomial_monomial(m, a), polynomial_monomial(n, b)) =
        polynomial_monomial(m + n, a * b)
} by {
    forall(k: Nat) {
        polynomial_mul_coeff_apply(polynomial_monomial(m, a), polynomial_monomial(n, b), k)
        polynomial_mul(polynomial_monomial(m, a), polynomial_monomial(n, b)).coeff(k) =
            polynomial_mul_coeff(polynomial_monomial(m, a), polynomial_monomial(n, b), k)
        mul_coeff_eq_range_sum(polynomial_monomial(m, a), polynomial_monomial(n, b), k)
        polynomial_mul_coeff(polynomial_monomial(m, a), polynomial_monomial(n, b), k) =
            range_sum(polynomial_mul_term_coeff(polynomial_monomial(m, a),
                polynomial_monomial(n, b), k), k.suc)
        if k = m + n {
            k = m + n
            forall(i: Nat) {
                if i < k.suc {
                    if i = m {
                        k - i = k - m
                        monomial_nat_lte_add_left(m, n)
                        m <= m + n
                        m <= k
                        add_sub(k, m)
                        k - m + m = k
                        k = m + n
                        k - m + m = m + n
                        n + m = m + n
                        k - m + m = n + m
                        add_cancels_right(k - m, m, n)
                        k - m = n
                        k - i = n
                        monomial_coeff_self(n, b)
                        polynomial_monomial(n, b).coeff(k - i) = b
                        monomial_coeff_self(m, a)
                        polynomial_monomial(m, a).coeff(i) = a
                        polynomial_mul_term_coeff(polynomial_monomial(m, a),
                            polynomial_monomial(n, b), k, i) =
                            polynomial_monomial(m, a).coeff(i) *
                            polynomial_monomial(n, b).coeff(k - i)
                        polynomial_mul_term_coeff(polynomial_monomial(m, a),
                            polynomial_monomial(n, b), k, i) = a * b
                        point_fn_at(m, a * b)
                        point_fn(m, a * b)(i) = a * b
                        polynomial_mul_term_coeff(polynomial_monomial(m, a),
                            polynomial_monomial(n, b), k)(i) =
                            point_fn(m, a * b)(i)
                    } else {
                        i != m
                        monomial_coeff_of_ne(m, a, i)
                        polynomial_monomial(m, a).coeff(i) = R.0
                        polynomial_mul_term_coeff(polynomial_monomial(m, a),
                            polynomial_monomial(n, b), k, i) =
                            polynomial_monomial(m, a).coeff(i) *
                            polynomial_monomial(n, b).coeff(k - i)
                        polynomial_mul_term_coeff(polynomial_monomial(m, a),
                            polynomial_monomial(n, b), k, i) = R.0 * polynomial_monomial(n, b).coeff(k - i)
                        R.0 * polynomial_monomial(n, b).coeff(k - i) = R.0
                        point_fn_off(m, a * b, i)
                        point_fn(m, a * b)(i) = R.0
                        polynomial_mul_term_coeff(polynomial_monomial(m, a),
                            polynomial_monomial(n, b), k)(i) =
                            point_fn(m, a * b)(i)
                    }
                }
            }
            range_sum_congr(polynomial_mul_term_coeff(polynomial_monomial(m, a),
                polynomial_monomial(n, b), k), point_fn(m, a * b), k.suc)
            range_sum(polynomial_mul_term_coeff(polynomial_monomial(m, a),
                polynomial_monomial(n, b), k), k.suc) =
                range_sum(point_fn(m, a * b), k.suc)
            lt_suc(m + n)
            m + n < (m + n).suc
            monomial_nat_lte_add_left(m, n)
            m <= m + n
            lte_and_lt(m, m + n, (m + n).suc)
            m < (m + n).suc
            k = m + n
            k.suc = (m + n).suc
            m < k.suc
            range_sum_point_fn(m, a * b, k.suc)
            range_sum(point_fn(m, a * b), k.suc) = a * b
            polynomial_mul_coeff(polynomial_monomial(m, a), polynomial_monomial(n, b), k) = a * b
            polynomial_mul(polynomial_monomial(m, a), polynomial_monomial(n, b)).coeff(k) = a * b
            monomial_coeff_self(m + n, a * b)
            polynomial_monomial(m + n, a * b).coeff(k) = a * b
            polynomial_mul(polynomial_monomial(m, a), polynomial_monomial(n, b)).coeff(k) =
                polynomial_monomial(m + n, a * b).coeff(k)
        } else {
            k != m + n
            forall(i: Nat) {
                if i < k.suc {
                    monomial_mul_coeff_vanishes_point(m, a, n, b, k, i)
                    polynomial_mul_term_coeff(polynomial_monomial(m, a),
                        polynomial_monomial(n, b), k, i) = R.0
                }
            }
            range_sum_zero_of_vanishes(polynomial_mul_term_coeff(polynomial_monomial(m, a),
                polynomial_monomial(n, b), k), k.suc)
            range_sum(polynomial_mul_term_coeff(polynomial_monomial(m, a),
                polynomial_monomial(n, b), k), k.suc) = R.0
            polynomial_mul_coeff(polynomial_monomial(m, a), polynomial_monomial(n, b), k) = R.0
            polynomial_mul(polynomial_monomial(m, a), polynomial_monomial(n, b)).coeff(k) = R.0
            monomial_coeff_of_ne(m + n, a * b, k)
            polynomial_monomial(m + n, a * b).coeff(k) = R.0
            polynomial_mul(polynomial_monomial(m, a), polynomial_monomial(n, b)).coeff(k) =
                polynomial_monomial(m + n, a * b).coeff(k)
        }
    }
    polynomial_ext_pointwise(polynomial_mul(polynomial_monomial(m, a), polynomial_monomial(n, b)),
        polynomial_monomial(m + n, a * b))
}

/// The constant polynomial is a monomial of exponent zero, so multiplying by
/// a monomial shifts only the coefficient.
theorem polynomial_constant_mul_monomial[R: CommRing](r: R, n: Nat, s: R) {
    polynomial_mul(polynomial_constant(r), polynomial_monomial(n, s)) =
        polynomial_monomial(n, r * s)
} by {
    polynomial_constant_eq_monomial_zero(r)
    polynomial_constant(r) = polynomial_monomial(Nat.0, r)
    polynomial_monomial_mul(Nat.0, r, n, s)
    polynomial_mul(polynomial_monomial(Nat.0, r), polynomial_monomial(n, s)) =
        polynomial_monomial(Nat.0 + n, r * s)
    polynomial_mul(polynomial_constant(r), polynomial_monomial(n, s)) =
        polynomial_monomial(Nat.0 + n, r * s)
    Nat.0 + n = n
    polynomial_monomial(Nat.0 + n, r * s) = polynomial_monomial(n, r * s)
    polynomial_mul(polynomial_constant(r), polynomial_monomial(n, s)) =
        polynomial_monomial(n, r * s)
}

/// Multiplying by a constant on the right shifts only the coefficient.
theorem polynomial_monomial_mul_constant[R: CommRing](n: Nat, s: R, r: R) {
    polynomial_mul(polynomial_monomial(n, s), polynomial_constant(r)) =
        polynomial_monomial(n, s * r)
} by {
    polynomial_constant_eq_monomial_zero(r)
    polynomial_constant(r) = polynomial_monomial(Nat.0, r)
    polynomial_monomial_mul(n, s, Nat.0, r)
    polynomial_mul(polynomial_monomial(n, s), polynomial_monomial(Nat.0, r)) =
        polynomial_monomial(n + Nat.0, s * r)
    polynomial_mul(polynomial_monomial(n, s), polynomial_constant(r)) =
        polynomial_monomial(n + Nat.0, s * r)
    n + Nat.0 = n
    polynomial_monomial(n + Nat.0, s * r) = polynomial_monomial(n, s * r)
    polynomial_mul(polynomial_monomial(n, s), polynomial_constant(r)) =
        polynomial_monomial(n, s * r)
}

/// A monomial is the zero polynomial exactly when its coefficient is zero.
theorem polynomial_monomial_eq_zero[R: Semiring](n: Nat, r: R) {
    (polynomial_monomial(n, r) = Polynomial[R].zero) = (r = R.0)
} by {
    if polynomial_monomial(n, r) = Polynomial[R].zero {
        polynomial_eq_coeff_at(polynomial_monomial(n, r), Polynomial[R].zero, n)
        polynomial_monomial(n, r).coeff(n) = Polynomial[R].zero.coeff(n)
        monomial_coeff_self(n, r)
        polynomial_monomial(n, r).coeff(n) = r
        polynomial_zero_coeff[R](n)
        Polynomial[R].zero.coeff(n) = R.0
        r = R.0
    }
    if r = R.0 {
        forall(k: Nat) {
            if k = n {
                monomial_coeff_self(n, r)
                polynomial_monomial(n, r).coeff(k) = r
                r = R.0
                polynomial_monomial(n, r).coeff(k) = R.0
                polynomial_zero_coeff[R](k)
                polynomial_monomial(n, r).coeff(k) = Polynomial[R].zero.coeff(k)
            } else {
                k != n
                monomial_coeff_of_ne(n, r, k)
                polynomial_monomial(n, r).coeff(k) = R.0
                polynomial_zero_coeff[R](k)
                polynomial_monomial(n, r).coeff(k) = Polynomial[R].zero.coeff(k)
            }
        }
        polynomial_ext_pointwise(polynomial_monomial(n, r), Polynomial[R].zero)
        polynomial_monomial(n, r) = Polynomial[R].zero
    }
}

/// A monomial with nonzero coefficient is not the zero polynomial.
theorem polynomial_monomial_ne_zero_of_ne[R: Semiring](n: Nat, r: R) {
    r != R.0 implies polynomial_monomial(n, r) != Polynomial[R].zero
} by {
    if r != R.0 {
        if polynomial_monomial(n, r) = Polynomial[R].zero {
            polynomial_monomial_eq_zero(n, r)
            r = R.0
            false
        }
        polynomial_monomial(n, r) != Polynomial[R].zero
    }
}

/// A constant polynomial with nonzero coefficient is not the zero polynomial.
theorem polynomial_constant_ne_zero_of_ne[R: Semiring](r: R) {
    r != R.0 implies polynomial_constant(r) != Polynomial[R].zero
} by {
    if r != R.0 {
        polynomial_constant_eq_monomial_zero(r)
        polynomial_constant(r) = polynomial_monomial(Nat.0, r)
        polynomial_monomial_ne_zero_of_ne(Nat.0, r)
        polynomial_monomial(Nat.0, r) != Polynomial[R].zero
        polynomial_constant(r) != Polynomial[R].zero
    }
}

/// Products of monomials are commutative.
theorem polynomial_monomial_mul_comm[R: CommRing](m: Nat, a: R, n: Nat, b: R) {
    polynomial_mul(polynomial_monomial(m, a), polynomial_monomial(n, b)) =
        polynomial_mul(polynomial_monomial(n, b), polynomial_monomial(m, a))
} by {
    polynomial_monomial_mul(m, a, n, b)
    polynomial_mul(polynomial_monomial(m, a), polynomial_monomial(n, b)) =
        polynomial_monomial(m + n, a * b)
    polynomial_monomial_mul(n, b, m, a)
    polynomial_mul(polynomial_monomial(n, b), polynomial_monomial(m, a)) =
        polynomial_monomial(n + m, b * a)
    m + n = n + m
    CommSemigroup.commutative[R](a, b)
    a * b = b * a
    polynomial_monomial(m + n, a * b) = polynomial_monomial(n + m, b * a)
    polynomial_mul(polynomial_monomial(m, a), polynomial_monomial(n, b)) =
        polynomial_mul(polynomial_monomial(n, b), polynomial_monomial(m, a))
}

/// Powers of the indeterminate multiply: `X^m * X^n = X^(m + n)`.
theorem polynomial_x_pow_mul[R: CommRing](m: Nat, n: Nat) {
    polynomial_mul(polynomial_monomial(m, R.1), polynomial_monomial(n, R.1)) =
        polynomial_monomial(m + n, R.1)
} by {
    polynomial_monomial_mul(m, R.1, n, R.1)
    polynomial_mul(polynomial_monomial(m, R.1), polynomial_monomial(n, R.1)) =
        polynomial_monomial(m + n, R.1 * R.1)
    R.1 * R.1 = R.1
    polynomial_monomial(m + n, R.1 * R.1) = polynomial_monomial(m + n, R.1)
    polynomial_mul(polynomial_monomial(m, R.1), polynomial_monomial(n, R.1)) =
        polynomial_monomial(m + n, R.1)
}

/// The value of `X^n` at a point is `x^n`.
theorem polynomial_x_pow_eval[R: CommRing](n: Nat, x: R) {
    polynomial_eval(polynomial_monomial(n, R.1), x) = x.pow(n)
} by {
    polynomial_eval_monomial(n, R.1, x)
    polynomial_eval(polynomial_monomial(n, R.1), x) = R.1 * x.pow(n)
    R.1 * x.pow(n) = x.pow(n)
}

/// The power `X^n` for positive `n` vanishes at zero.
theorem polynomial_x_pow_zero_eval[R: CommRing](n: Nat) {
    polynomial_eval(polynomial_monomial(n.suc, R.1), R.0) = R.0
} by {
    polynomial_eval_monomial(n.suc, R.1, R.0)
    polynomial_eval(polynomial_monomial(n.suc, R.1), R.0) = R.1 * R.0.pow(n.suc)
    R.0.pow(n.suc) = R.0 * R.0.pow(n)
    R.1 * (R.0 * R.0.pow(n)) = R.0 * R.0.pow(n)
    R.0 * R.0.pow(n) = R.0
    R.1 * R.0.pow(n.suc) = R.0
    polynomial_eval(polynomial_monomial(n.suc, R.1), R.0) = R.0
}
