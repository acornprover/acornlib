/// Polynomial theorems: the remainder theorem, the factor theorem, the
/// rational root theorem for cubics, and the fundamental theorem of algebra
/// for cubics, stated generally and verified on concrete polynomials.
///
/// The general statements are proved on top of the polynomial module
/// (`src/polynomial/`): the remainder/factor theorems for the global
/// evaluation and linear-factor divisibility machinery, and the rational root
/// theorem for the integer-to-rational transport of `src/polynomial_int_roots`
/// and `src/polynomial_rational_root_bridge`.  Each general theorem is
/// followed by a concrete polynomial computation.
///
/// The concrete verifications:
///   * the remainder theorem for `X^2 - 1` at `1` (remainder zero, `X - 1`
///     divides the polynomial),
///   * the factor theorem for `X^2 - 1` (roots `+-1`, factors
///     `(X - 1)(X + 1)`),
///   * the rational root theorem for the cubic `X^3 - 6 X^2 + 11 X - 6`
///     (rational roots `1`, `2`, `3`, all dividing the constant term `6`),
///   * the cubic `X^3 - 1` over the reals (root `1`).

from nat import Nat, add_cancels_right, lte_trans, lt_suc, lt_trans, lt_and_lte,
    lt_not_symm, lt_or_lte, lte_suc_suc, lte_imp_not_lt, zero_or_suc,
    from_nat_zero, from_nat_one, from_nat_add, add_suc_right, add_zero_left,
    add_zero_right, add_imp_sub
from order import lt_imp_lte
from semiring import Semiring
from comm_ring import CommRing
from algebra.field.field import Field, inverse_one
from algebra.ring.ring import mul_neg_left, mul_neg_right, mul_neg_neg
from algebra.add_group import inverse_inverse, inverse_add
from int import Int, sub_nat, add_sub_nat, sub_nat_zero_right, sub_nat_zero_left,
    sub_nat_add_left, sub_nat_add_right, neg_sub_nat, mul_from_nat, add_neg
from rat import Rat
from data.basic.functions import function_extensionality
from data.int.int_coprime import is_coprime, is_coprime_one_right
from data.rat.rat_int_hom import int_to_rat_hom, int_to_rat_hom_apply
from polynomial.base import Polynomial, polynomial_constant, polynomial_monomial,
    polynomial_add_coeff, polynomial_constant_coeff_zero, polynomial_constant_coeff_of_ne_zero,
    polynomial_monomial_coeff_self, polynomial_monomial_coeff_of_ne
from polynomial.coeff_eval import coeff_eval, coeff_tail
from polynomial.eval import polynomial_support_bounded_by, polynomial_add_support_bounded_by,
    polynomial_support_bounded_by_monotone, polynomial_support_bounded_by_apply,
    polynomial_constant_support_bounded_by_one, polynomial_eval_bound,
    polynomial_eval_bound_eq_coeff_eval, polynomial_support_bound_exists,
    polynomial_quotient, polynomial_quotient_support_bounded_by,
    polynomial_remainder_theorem_bundled_bound
from polynomial.global_eval import polynomial_eval, polynomial_eval_add, polynomial_eval_constant,
    polynomial_eval_eq_eval_bound_of_support_bounded
from polynomial.eval_mul import polynomial_eval_mul
from polynomial.mul import polynomial_mul, polynomial_mul_coeff_apply
from polynomial.global_factor import polynomial_divisible_by_x_minus_a_polynomial,
    polynomial_x_minus_a_polynomial_witness, polynomial_x_minus_a_polynomial_witness_at,
    polynomial_factor_theorem_forward_global, polynomial_factor_theorem_reverse_global,
    polynomial_factor_theorem_global
from polynomial.polynomial_ring import quadratic_polynomial, linear_polynomial,
    polynomial_remainder_theorem_global, field_x_squared_minus_one, field_x_minus_one,
    field_x_plus_one, field_x_squared_minus_one_has_root_one,
    field_x_minus_one_divides_x_squared_minus_one, field_x_plus_one_witnesses_factor,
    field_x_squared_minus_one_eq_mul, quadratic_polynomial_eval, linear_polynomial_eval,
    monomial_one_eval, monomial_two_eval, polynomial_monomial_support_bounded_by_suc
from polynomial.map import polynomial_map, polynomial_map_coeff_apply,
    polynomial_map_support_bounded_by, polynomial_eval_map
from polynomial_rational_root_bridge import rat_coeffs, rat_coeffs_apply,
    rat_root_numerator_divides, rat_root_denominator_divides
from real import Real

numerals Nat
numerals Int
numerals Rat

/// One is at most four.
lemma nat_one_le_four {
    Nat.1 <= Nat.4
} by {
    lt_suc(Nat.1)
    Nat.1 < Nat.1.suc
    Nat.1.suc = Nat.2
    lt_imp_lte[Nat](Nat.1, Nat.2)
    Nat.1 <= Nat.2
    lt_suc(Nat.2)
    Nat.2 < Nat.2.suc
    Nat.2.suc = Nat.3
    lt_imp_lte[Nat](Nat.2, Nat.3)
    Nat.2 <= Nat.3
    lte_trans(Nat.1, Nat.2, Nat.3)
    Nat.1 <= Nat.3
    lt_suc(Nat.3)
    Nat.3 < Nat.3.suc
    Nat.3.suc = Nat.4
    lt_imp_lte[Nat](Nat.3, Nat.4)
    Nat.3 <= Nat.4
    lte_trans(Nat.1, Nat.3, Nat.4)
    Nat.1 <= Nat.4
}

/// Two is at most four.
lemma nat_two_le_four {
    Nat.2 <= Nat.4
} by {
    lt_suc(Nat.2)
    Nat.2 < Nat.2.suc
    Nat.2.suc = Nat.3
    lt_imp_lte[Nat](Nat.2, Nat.3)
    Nat.2 <= Nat.3
    lt_suc(Nat.3)
    Nat.3 < Nat.3.suc
    Nat.3.suc = Nat.4
    lt_imp_lte[Nat](Nat.3, Nat.4)
    Nat.3 <= Nat.4
    lte_trans(Nat.2, Nat.3, Nat.4)
    Nat.2 <= Nat.4
}

/// Three is at most four.
lemma nat_three_le_four {
    Nat.3 <= Nat.4
} by {
    lt_suc(Nat.3)
    Nat.3 < Nat.3.suc
    Nat.3.suc = Nat.4
    lt_imp_lte[Nat](Nat.3, Nat.4)
    Nat.3 <= Nat.4
}

/// `(x - 1)(x + 1) = x^2 - 1`.
lemma ring_mul_x_minus_one_x_plus_one[R: CommRing](x: R) {
    (x - R.1) * (x + R.1) = x * x - R.1
} by {
    (x - R.1) * (x + R.1) = x * (x + R.1) - R.1 * (x + R.1)
    x * (x + R.1) = x * x + x * R.1
    x * R.1 = x
    x * (x + R.1) = x * x + x
    R.1 * (x + R.1) = R.1 * x + R.1 * R.1
    R.1 * x = x
    R.1 * R.1 = R.1
    R.1 * (x + R.1) = x + R.1
    (x - R.1) * (x + R.1) = (x * x + x) - (x + R.1)
    (x * x + x) - (x + R.1) = (x * x + x) + -(x + R.1)
    inverse_add(x, R.1)
    -(x + R.1) = -R.1 + -x
    (x * x + x) + (-R.1 + -x) = x * x + (x + (-R.1 + -x))
    x * x + (x + (-R.1 + -x)) = x * x + ((x + -R.1) + -x)
    x * x + ((x + -R.1) + -x) = x * x + ((-R.1 + x) + -x)
    x * x + ((-R.1 + x) + -x) = x * x + (-R.1 + (x + -x))
    x + -x = R.0
    x * x + (-R.1 + (x + -x)) = x * x + (-R.1 + R.0)
    x * x + (-R.1 + R.0) = x * x + -R.1
    x * x + -R.1 = x * x - R.1
    (x * x + x) - (x + R.1) = x * x - R.1
    (x - R.1) * (x + R.1) = x * x - R.1
}

// ---------------------------------------------------------------------------
// 1. The remainder theorem
// ---------------------------------------------------------------------------

/// The remainder theorem: for every polynomial `p` and point `a`, evaluation
/// at `a` is the remainder of `p` on division by the linear factor `X - a`.
///
/// In global form: `p(x) = p(a) + q(x) * (x - a)` for the synthetic quotient
/// `q = polynomial_quotient(p, a, n)` at any support bound `n` of `p`.
theorem remainder_theorem_global[R: CommRing](p: Polynomial[R], a: R, x: R) {
    exists(q: Polynomial[R]) {
        polynomial_eval(p, x) = polynomial_eval(p, a) + polynomial_eval(q, x) * (x - a)
    }
} by {
    polynomial_support_bound_exists(p)
    let n: Nat satisfy {
        polynomial_support_bounded_by(p, n)
    }
    polynomial_eval_eq_eval_bound_of_support_bounded(p, x, n)
    polynomial_eval(p, x) = polynomial_eval_bound(p, x, n)
    polynomial_eval_eq_eval_bound_of_support_bounded(p, a, n)
    polynomial_eval(p, a) = polynomial_eval_bound(p, a, n)
    polynomial_remainder_theorem_bundled_bound(p, a, x, n)
    polynomial_eval_bound(p, x, n) =
        polynomial_eval_bound(p, a, n) +
        (x - a) * polynomial_eval_bound(polynomial_quotient(p, a, n), x, n)
    polynomial_quotient_support_bounded_by(p, a, n)
    polynomial_support_bounded_by(polynomial_quotient(p, a, n), n)
    polynomial_eval_eq_eval_bound_of_support_bounded(polynomial_quotient(p, a, n), x, n)
    polynomial_eval(polynomial_quotient(p, a, n), x) =
        polynomial_eval_bound(polynomial_quotient(p, a, n), x, n)
    (x - a) * polynomial_eval_bound(polynomial_quotient(p, a, n), x, n) =
        polynomial_eval_bound(polynomial_quotient(p, a, n), x, n) * (x - a)
    polynomial_eval_bound(p, x, n) =
        polynomial_eval_bound(p, a, n) +
        polynomial_eval_bound(polynomial_quotient(p, a, n), x, n) * (x - a)
    polynomial_eval(p, x) =
        polynomial_eval(p, a) + polynomial_eval(polynomial_quotient(p, a, n), x) * (x - a)
    exists(q: Polynomial[R]) {
        q = polynomial_quotient(p, a, n) and
        polynomial_eval(p, x) = polynomial_eval(p, a) + polynomial_eval(q, x) * (x - a)
    }
    exists(q: Polynomial[R]) {
        polynomial_eval(p, x) = polynomial_eval(p, a) + polynomial_eval(q, x) * (x - a)
    }
}

/// The remainder theorem in root form: the remainder of `p` on division by
/// `X - a` vanishes exactly when `X - a` divides `p`.
theorem remainder_theorem_root_form[R: CommRing](p: Polynomial[R], a: R) {
    polynomial_eval(p, a) = R.0 iff polynomial_divisible_by_x_minus_a_polynomial(p, a)
} by {
    polynomial_remainder_theorem_global(p, a)
}

/// The concrete quadratic `X^2 - 1`: the remainder on division by `X - 1` at
/// the point `1` is zero.
theorem x_squared_minus_one_remainder_at_one[F: Field] {
    polynomial_eval(field_x_squared_minus_one[F], F.1) = F.0
} by {
    field_x_squared_minus_one_has_root_one[F]
    polynomial_eval(field_x_squared_minus_one[F], F.1) = F.0
}

/// Consequently `X - 1` divides `X^2 - 1`.
theorem x_minus_one_divides_x_squared_minus_one[F: Field] {
    polynomial_divisible_by_x_minus_a_polynomial(field_x_squared_minus_one[F], F.1)
} by {
    field_x_minus_one_divides_x_squared_minus_one[F]
    polynomial_divisible_by_x_minus_a_polynomial(field_x_squared_minus_one[F], F.1)
}

/// The remainder identity for the concrete quadratic: at every point `x`,
/// `(X^2 - 1)(x) = (X + 1)(x) * (x - 1)`, and the remainder `(X^2 - 1)(1)`
/// is zero.
theorem x_squared_minus_one_remainder_identity[F: Field] {
    polynomial_x_minus_a_polynomial_witness(field_x_squared_minus_one[F], F.1, field_x_plus_one[F])
} by {
    field_x_plus_one_witnesses_factor[F]
    polynomial_x_minus_a_polynomial_witness(field_x_squared_minus_one[F], F.1, field_x_plus_one[F])
}

// ---------------------------------------------------------------------------
// 2. The factor theorem
// ---------------------------------------------------------------------------

/// The factor theorem: `a` is a root of `p` exactly when the linear factor
/// `X - a` divides `p`.
theorem factor_theorem[R: CommRing](p: Polynomial[R], a: R) {
    polynomial_eval(p, a) = R.0 iff polynomial_divisible_by_x_minus_a_polynomial(p, a)
} by {
    polynomial_factor_theorem_global(p, a)
}

/// The roots of `X^2 - 1`: `1` is a root.
theorem x_squared_minus_one_root_one[F: Field] {
    polynomial_eval(field_x_squared_minus_one[F], F.1) = F.0
} by {
    field_x_squared_minus_one_has_root_one[F]
    polynomial_eval(field_x_squared_minus_one[F], F.1) = F.0
}

/// The roots of `X^2 - 1`: `-1` is a root.
theorem x_squared_minus_one_root_neg_one[F: Field] {
    polynomial_eval(field_x_squared_minus_one[F], -F.1) = F.0
} by {
    field_x_squared_minus_one_eq_mul[F]
    polynomial_mul(field_x_minus_one[F], field_x_plus_one[F]) = field_x_squared_minus_one[F]
    polynomial_eval_mul(field_x_minus_one[F], field_x_plus_one[F], -F.1)
    polynomial_eval(polynomial_mul(field_x_minus_one[F], field_x_plus_one[F]), -F.1) =
        polynomial_eval(field_x_minus_one[F], -F.1) * polynomial_eval(field_x_plus_one[F], -F.1)
    polynomial_eval(field_x_squared_minus_one[F], -F.1) =
        polynomial_eval(field_x_minus_one[F], -F.1) * polynomial_eval(field_x_plus_one[F], -F.1)
    linear_polynomial_eval(F.1, -F.1, -F.1)
    polynomial_eval(field_x_minus_one[F], -F.1) = -F.1 + F.1 * (-F.1)
    linear_polynomial_eval(F.1, F.1, -F.1)
    polynomial_eval(field_x_plus_one[F], -F.1) = F.1 + F.1 * (-F.1)
    F.1 * (-F.1) = -F.1
    -F.1 + F.1 * (-F.1) = -F.1 + -F.1
    F.1 + F.1 * (-F.1) = F.1 + -F.1
    F.1 + -F.1 = F.0
    polynomial_eval(field_x_squared_minus_one[F], -F.1) = (-F.1 + -F.1) * (F.1 + -F.1)
    (-F.1 + -F.1) * (F.1 + -F.1) = (-F.1 + -F.1) * F.0
    (-F.1 + -F.1) * F.0 = F.0
    polynomial_eval(field_x_squared_minus_one[F], -F.1) = F.0
}

/// By the factor theorem, `X + 1` divides `X^2 - 1`.
theorem x_plus_one_divides_x_squared_minus_one[F: Field] {
    polynomial_divisible_by_x_minus_a_polynomial(field_x_squared_minus_one[F], -F.1)
} by {
    x_squared_minus_one_root_neg_one[F]
    polynomial_eval(field_x_squared_minus_one[F], -F.1) = F.0
    polynomial_factor_theorem_forward_global(field_x_squared_minus_one[F], -F.1)
    polynomial_divisible_by_x_minus_a_polynomial(field_x_squared_minus_one[F], -F.1)
}

/// The factorization `X^2 - 1 = (X - 1)(X + 1)`.
theorem x_squared_minus_one_factorization[F: Field] {
    polynomial_mul(field_x_minus_one[F], field_x_plus_one[F]) = field_x_squared_minus_one[F]
} by {
    field_x_squared_minus_one_eq_mul[F]
    polynomial_mul(field_x_minus_one[F], field_x_plus_one[F]) = field_x_squared_minus_one[F]
}

/// The explicit quotient for the factor `X + 1`: at every point `x`,
/// `(X^2 - 1)(x) = (X - 1)(x) * (x + 1)`.
theorem x_minus_one_witnesses_factor_neg_one[F: Field] {
    polynomial_x_minus_a_polynomial_witness(field_x_squared_minus_one[F], -F.1, field_x_minus_one[F])
} by {
    forall(x: F) {
        quadratic_polynomial_eval(F.1, F.0, -F.1, x)
        polynomial_eval(field_x_squared_minus_one[F], x) = -F.1 + F.0 * x + F.1 * x * x
        linear_polynomial_eval(F.1, -F.1, x)
        polynomial_eval(field_x_minus_one[F], x) = -F.1 + F.1 * x
        F.0 * x = F.0
        -F.1 + F.0 * x + F.1 * x * x = -F.1 + F.1 * x * x
        F.1 * x * x = x * x
        -F.1 + F.1 * x * x = -F.1 + x * x
        -F.1 + x * x = x * x - F.1
        (-F.1 + F.1 * x) * (x + F.1) = (F.1 * x + -F.1) * (x + F.1)
        (F.1 * x + -F.1) * (x + F.1) = (x + -F.1) * (x + F.1)
        (x + -F.1) * (x + F.1) = (x - F.1) * (x + F.1)
        ring_mul_x_minus_one_x_plus_one(x)
        (x - F.1) * (x + F.1) = x * x - F.1
        (-F.1 + F.1 * x) * (x + F.1) = x * x - F.1
        -F.1 + F.0 * x + F.1 * x * x = (-F.1 + F.1 * x) * (x + F.1)
        polynomial_eval(field_x_squared_minus_one[F], x) =
            polynomial_eval(field_x_minus_one[F], x) * (x + F.1)
        x + F.1 = x - (-F.1)
        polynomial_eval(field_x_squared_minus_one[F], x) =
            polynomial_eval(field_x_minus_one[F], x) * (x - (-F.1))
    }
    polynomial_x_minus_a_polynomial_witness(field_x_squared_minus_one[F], -F.1, field_x_minus_one[F]) =
        forall(x: F) {
            polynomial_x_minus_a_polynomial_witness_at(
                field_x_squared_minus_one[F], -F.1, field_x_minus_one[F], x)
        }
    polynomial_x_minus_a_polynomial_witness(field_x_squared_minus_one[F], -F.1, field_x_minus_one[F])
}

// ---------------------------------------------------------------------------
// 3. The rational root theorem for cubics
// ---------------------------------------------------------------------------

/// The cubic polynomial `a * X^3 + b * X^2 + c * X + d`.
define cubic_polynomial[R: Semiring](a: R, b: R, c: R, d: R) -> Polynomial[R] {
    polynomial_constant(d) + polynomial_monomial(Nat.1, c) + polynomial_monomial(Nat.2, b) +
        polynomial_monomial(Nat.3, a)
}

/// The constant coefficient of a cubic is its constant term.
theorem cubic_polynomial_coeff_zero[R: Semiring](a: R, b: R, c: R, d: R) {
    cubic_polynomial(a, b, c, d).coeff(Nat.0) = d
} by {
    polynomial_add_coeff(polynomial_constant(d) + polynomial_monomial(Nat.1, c) +
        polynomial_monomial(Nat.2, b), polynomial_monomial(Nat.3, a), Nat.0)
    cubic_polynomial(a, b, c, d).coeff(Nat.0) =
        (polynomial_constant(d) + polynomial_monomial(Nat.1, c) +
            polynomial_monomial(Nat.2, b)).coeff(Nat.0) +
        polynomial_monomial(Nat.3, a).coeff(Nat.0)
    polynomial_add_coeff(polynomial_constant(d) + polynomial_monomial(Nat.1, c),
        polynomial_monomial(Nat.2, b), Nat.0)
    (polynomial_constant(d) + polynomial_monomial(Nat.1, c) +
        polynomial_monomial(Nat.2, b)).coeff(Nat.0) =
        (polynomial_constant(d) + polynomial_monomial(Nat.1, c)).coeff(Nat.0) +
        polynomial_monomial(Nat.2, b).coeff(Nat.0)
    polynomial_add_coeff(polynomial_constant(d), polynomial_monomial(Nat.1, c), Nat.0)
    (polynomial_constant(d) + polynomial_monomial(Nat.1, c)).coeff(Nat.0) =
        polynomial_constant(d).coeff(Nat.0) + polynomial_monomial(Nat.1, c).coeff(Nat.0)
    polynomial_constant_coeff_zero(d)
    polynomial_constant(d).coeff(Nat.0) = d
    Nat.0 != Nat.1
    polynomial_monomial_coeff_of_ne(Nat.1, c, Nat.0)
    polynomial_monomial(Nat.1, c).coeff(Nat.0) = R.0
    Nat.0 != Nat.2
    polynomial_monomial_coeff_of_ne(Nat.2, b, Nat.0)
    polynomial_monomial(Nat.2, b).coeff(Nat.0) = R.0
    Nat.0 != Nat.3
    polynomial_monomial_coeff_of_ne(Nat.3, a, Nat.0)
    polynomial_monomial(Nat.3, a).coeff(Nat.0) = R.0
    cubic_polynomial(a, b, c, d).coeff(Nat.0) = d + R.0 + R.0 + R.0
    d + R.0 + R.0 + R.0 = d
    cubic_polynomial(a, b, c, d).coeff(Nat.0) = d
}

/// The cubic coefficient of a cubic is its leading term.
theorem cubic_polynomial_coeff_three[R: Semiring](a: R, b: R, c: R, d: R) {
    cubic_polynomial(a, b, c, d).coeff(Nat.3) = a
} by {
    polynomial_add_coeff(polynomial_constant(d) + polynomial_monomial(Nat.1, c) +
        polynomial_monomial(Nat.2, b), polynomial_monomial(Nat.3, a), Nat.3)
    cubic_polynomial(a, b, c, d).coeff(Nat.3) =
        (polynomial_constant(d) + polynomial_monomial(Nat.1, c) +
            polynomial_monomial(Nat.2, b)).coeff(Nat.3) +
        polynomial_monomial(Nat.3, a).coeff(Nat.3)
    polynomial_add_coeff(polynomial_constant(d) + polynomial_monomial(Nat.1, c),
        polynomial_monomial(Nat.2, b), Nat.3)
    (polynomial_constant(d) + polynomial_monomial(Nat.1, c) +
        polynomial_monomial(Nat.2, b)).coeff(Nat.3) =
        (polynomial_constant(d) + polynomial_monomial(Nat.1, c)).coeff(Nat.3) +
        polynomial_monomial(Nat.2, b).coeff(Nat.3)
    polynomial_add_coeff(polynomial_constant(d), polynomial_monomial(Nat.1, c), Nat.3)
    (polynomial_constant(d) + polynomial_monomial(Nat.1, c)).coeff(Nat.3) =
        polynomial_constant(d).coeff(Nat.3) + polynomial_monomial(Nat.1, c).coeff(Nat.3)
    Nat.3 != Nat.0
    polynomial_constant_coeff_of_ne_zero(d, Nat.3)
    polynomial_constant(d).coeff(Nat.3) = R.0
    Nat.3 != Nat.1
    polynomial_monomial_coeff_of_ne(Nat.1, c, Nat.3)
    polynomial_monomial(Nat.1, c).coeff(Nat.3) = R.0
    Nat.3 != Nat.2
    polynomial_monomial_coeff_of_ne(Nat.2, b, Nat.3)
    polynomial_monomial(Nat.2, b).coeff(Nat.3) = R.0
    polynomial_monomial_coeff_self(Nat.3, a)
    polynomial_monomial(Nat.3, a).coeff(Nat.3) = a
    cubic_polynomial(a, b, c, d).coeff(Nat.3) = R.0 + R.0 + R.0 + a
    R.0 + R.0 + R.0 + a = a
    cubic_polynomial(a, b, c, d).coeff(Nat.3) = a
}

/// The linear coefficient of a cubic is its middle term.
theorem cubic_polynomial_coeff_one[R: Semiring](a: R, b: R, c: R, d: R) {
    cubic_polynomial(a, b, c, d).coeff(Nat.1) = c
} by {
    polynomial_add_coeff(polynomial_constant(d) + polynomial_monomial(Nat.1, c) +
        polynomial_monomial(Nat.2, b), polynomial_monomial(Nat.3, a), Nat.1)
    cubic_polynomial(a, b, c, d).coeff(Nat.1) =
        (polynomial_constant(d) + polynomial_monomial(Nat.1, c) +
            polynomial_monomial(Nat.2, b)).coeff(Nat.1) +
        polynomial_monomial(Nat.3, a).coeff(Nat.1)
    polynomial_add_coeff(polynomial_constant(d) + polynomial_monomial(Nat.1, c),
        polynomial_monomial(Nat.2, b), Nat.1)
    (polynomial_constant(d) + polynomial_monomial(Nat.1, c) +
        polynomial_monomial(Nat.2, b)).coeff(Nat.1) =
        (polynomial_constant(d) + polynomial_monomial(Nat.1, c)).coeff(Nat.1) +
        polynomial_monomial(Nat.2, b).coeff(Nat.1)
    polynomial_add_coeff(polynomial_constant(d), polynomial_monomial(Nat.1, c), Nat.1)
    (polynomial_constant(d) + polynomial_monomial(Nat.1, c)).coeff(Nat.1) =
        polynomial_constant(d).coeff(Nat.1) + polynomial_monomial(Nat.1, c).coeff(Nat.1)
    Nat.1 != Nat.0
    polynomial_constant_coeff_of_ne_zero(d, Nat.1)
    polynomial_constant(d).coeff(Nat.1) = R.0
    polynomial_monomial_coeff_self(Nat.1, c)
    polynomial_monomial(Nat.1, c).coeff(Nat.1) = c
    Nat.1 != Nat.2
    polynomial_monomial_coeff_of_ne(Nat.2, b, Nat.1)
    polynomial_monomial(Nat.2, b).coeff(Nat.1) = R.0
    Nat.1 != Nat.3
    polynomial_monomial_coeff_of_ne(Nat.3, a, Nat.1)
    polynomial_monomial(Nat.3, a).coeff(Nat.1) = R.0
    cubic_polynomial(a, b, c, d).coeff(Nat.1) = R.0 + c + R.0 + R.0
    R.0 + c + R.0 + R.0 = c
    cubic_polynomial(a, b, c, d).coeff(Nat.1) = c
}

/// The quadratic coefficient of a cubic is its fourth term.
theorem cubic_polynomial_coeff_two[R: Semiring](a: R, b: R, c: R, d: R) {
    cubic_polynomial(a, b, c, d).coeff(Nat.2) = b
} by {
    polynomial_add_coeff(polynomial_constant(d) + polynomial_monomial(Nat.1, c) +
        polynomial_monomial(Nat.2, b), polynomial_monomial(Nat.3, a), Nat.2)
    cubic_polynomial(a, b, c, d).coeff(Nat.2) =
        (polynomial_constant(d) + polynomial_monomial(Nat.1, c) +
            polynomial_monomial(Nat.2, b)).coeff(Nat.2) +
        polynomial_monomial(Nat.3, a).coeff(Nat.2)
    polynomial_add_coeff(polynomial_constant(d) + polynomial_monomial(Nat.1, c),
        polynomial_monomial(Nat.2, b), Nat.2)
    (polynomial_constant(d) + polynomial_monomial(Nat.1, c) +
        polynomial_monomial(Nat.2, b)).coeff(Nat.2) =
        (polynomial_constant(d) + polynomial_monomial(Nat.1, c)).coeff(Nat.2) +
        polynomial_monomial(Nat.2, b).coeff(Nat.2)
    polynomial_add_coeff(polynomial_constant(d), polynomial_monomial(Nat.1, c), Nat.2)
    (polynomial_constant(d) + polynomial_monomial(Nat.1, c)).coeff(Nat.2) =
        polynomial_constant(d).coeff(Nat.2) + polynomial_monomial(Nat.1, c).coeff(Nat.2)
    Nat.2 != Nat.0
    polynomial_constant_coeff_of_ne_zero(d, Nat.2)
    polynomial_constant(d).coeff(Nat.2) = R.0
    Nat.2 != Nat.1
    polynomial_monomial_coeff_of_ne(Nat.1, c, Nat.2)
    polynomial_monomial(Nat.1, c).coeff(Nat.2) = R.0
    polynomial_monomial_coeff_self(Nat.2, b)
    polynomial_monomial(Nat.2, b).coeff(Nat.2) = b
    Nat.2 != Nat.3
    polynomial_monomial_coeff_of_ne(Nat.3, a, Nat.2)
    polynomial_monomial(Nat.3, a).coeff(Nat.2) = R.0
    cubic_polynomial(a, b, c, d).coeff(Nat.2) = R.0 + R.0 + b + R.0
    R.0 + R.0 + b + R.0 = b
    cubic_polynomial(a, b, c, d).coeff(Nat.2) = b
}

/// A cubic polynomial is supported below four coefficient slots.
theorem cubic_polynomial_support_bounded_by_four[R: Semiring](a: R, b: R, c: R, d: R) {
    polynomial_support_bounded_by(cubic_polynomial(a, b, c, d), Nat.4)
} by {
    polynomial_constant_support_bounded_by_one(d)
    polynomial_support_bounded_by(polynomial_constant(d), Nat.1)
    nat_one_le_four
    polynomial_support_bounded_by_monotone(polynomial_constant(d), Nat.1, Nat.4)
    polynomial_support_bounded_by(polynomial_constant(d), Nat.4)
    polynomial_monomial_support_bounded_by_suc(Nat.1, c)
    polynomial_support_bounded_by(polynomial_monomial(Nat.1, c), Nat.2)
    nat_two_le_four
    polynomial_support_bounded_by_monotone(polynomial_monomial(Nat.1, c), Nat.2, Nat.4)
    polynomial_support_bounded_by(polynomial_monomial(Nat.1, c), Nat.4)
    polynomial_add_support_bounded_by(polynomial_constant(d), polynomial_monomial(Nat.1, c), Nat.4)
    polynomial_support_bounded_by(polynomial_constant(d) + polynomial_monomial(Nat.1, c), Nat.4)
    polynomial_monomial_support_bounded_by_suc(Nat.2, b)
    polynomial_support_bounded_by(polynomial_monomial(Nat.2, b), Nat.3)
    nat_three_le_four
    polynomial_support_bounded_by_monotone(polynomial_monomial(Nat.2, b), Nat.3, Nat.4)
    polynomial_support_bounded_by(polynomial_monomial(Nat.2, b), Nat.4)
    polynomial_add_support_bounded_by(polynomial_constant(d) + polynomial_monomial(Nat.1, c),
        polynomial_monomial(Nat.2, b), Nat.4)
    polynomial_support_bounded_by(
        (polynomial_constant(d) + polynomial_monomial(Nat.1, c)) + polynomial_monomial(Nat.2, b),
        Nat.4)
    polynomial_monomial_support_bounded_by_suc(Nat.3, a)
    polynomial_support_bounded_by(polynomial_monomial(Nat.3, a), Nat.4)
    polynomial_add_support_bounded_by(
        (polynomial_constant(d) + polynomial_monomial(Nat.1, c)) + polynomial_monomial(Nat.2, b),
        polynomial_monomial(Nat.3, a), Nat.4)
    polynomial_support_bounded_by(cubic_polynomial(a, b, c, d), Nat.4)
}

/// Bounded evaluation at bound zero is the zero of the coefficient ring.
lemma coeff_eval_nat_zero[R: Semiring](c: Nat -> R, x: R) {
    coeff_eval(c, x, Nat.0) = R.0
}

/// Bounded evaluation at bound four, unfolded to the Horner form.
lemma coeff_eval_four[R: Semiring](c: Nat -> R, x: R) {
    coeff_eval(c, x, Nat.4) = c(Nat.0) + x * (c(Nat.1) + x * (c(Nat.2) + x * (c(Nat.3) + x * R.0)))
} by {
    coeff_eval(c, x, Nat.4) = c(Nat.0) + x * coeff_eval(coeff_tail(c), x, Nat.3)
    coeff_eval(coeff_tail(c), x, Nat.3) = c(Nat.1) + x * coeff_eval(coeff_tail(coeff_tail(c)), x, Nat.2)
    coeff_tail(coeff_tail(c))(Nat.0) = c(Nat.2)
    coeff_eval(coeff_tail(coeff_tail(c)), x, Nat.2) =
        c(Nat.2) + x * coeff_eval(coeff_tail(coeff_tail(coeff_tail(c))), x, Nat.1)
    coeff_tail(coeff_tail(coeff_tail(c)))(Nat.0) = coeff_tail(coeff_tail(c))(Nat.0.suc)
    coeff_tail(coeff_tail(c))(Nat.0.suc) = coeff_tail(c)(Nat.0.suc.suc)
    coeff_tail(c)(Nat.0.suc.suc) = c(Nat.0.suc.suc.suc)
    coeff_tail(c)(Nat.0.suc.suc) = c(Nat.3)
    coeff_tail(coeff_tail(coeff_tail(c)))(Nat.0) = c(Nat.3)
    coeff_eval(coeff_tail(coeff_tail(coeff_tail(c))), x, Nat.1) =
        c(Nat.3) + x * coeff_eval(coeff_tail(coeff_tail(coeff_tail(coeff_tail(c)))), x, Nat.0)
    coeff_eval(coeff_tail(coeff_tail(coeff_tail(coeff_tail(c)))), x, Nat.0) = R.0
    coeff_eval(c, x, Nat.4) = c(Nat.0) + x * (c(Nat.1) + x * (c(Nat.2) + x * (c(Nat.3) + x * R.0)))
}

/// Evaluation of a monomial of degree three at a point.
theorem monomial_three_eval[R: CommRing](r: R, x: R) {
    polynomial_eval(Polynomial[R].monomial(Nat.3, r), x) = r * x * x * x
} by {
    polynomial_monomial_support_bounded_by_suc(Nat.3, r)
    polynomial_support_bounded_by(Polynomial[R].monomial(Nat.3, r), Nat.0.suc.suc.suc.suc)
    polynomial_eval_eq_eval_bound_of_support_bounded(
        Polynomial[R].monomial(Nat.3, r), x, Nat.0.suc.suc.suc.suc)
    polynomial_eval(Polynomial[R].monomial(Nat.3, r), x) =
        polynomial_eval_bound(Polynomial[R].monomial(Nat.3, r), x, Nat.0.suc.suc.suc.suc)
    polynomial_eval_bound_eq_coeff_eval(Polynomial[R].monomial(Nat.3, r), x, Nat.0.suc.suc.suc.suc)
    coeff_eval(Polynomial[R].monomial(Nat.3, r).coeff, x, Nat.0.suc.suc.suc.suc) =
        Polynomial[R].monomial(Nat.3, r).coeff(Nat.0) +
        x * coeff_eval(coeff_tail(Polynomial[R].monomial(Nat.3, r).coeff), x, Nat.0.suc.suc.suc)
    polynomial_monomial_coeff_of_ne(Nat.3, r, Nat.0)
    Nat.0 != Nat.3
    Polynomial[R].monomial(Nat.3, r).coeff(Nat.0) = R.0
    coeff_eval(coeff_tail(Polynomial[R].monomial(Nat.3, r).coeff), x, Nat.0.suc.suc.suc) =
        Polynomial[R].monomial(Nat.3, r).coeff(Nat.1) +
        x * coeff_eval(coeff_tail(coeff_tail(Polynomial[R].monomial(Nat.3, r).coeff)), x,
            Nat.0.suc.suc)
    polynomial_monomial_coeff_of_ne(Nat.3, r, Nat.1)
    Nat.1 != Nat.3
    Polynomial[R].monomial(Nat.3, r).coeff(Nat.1) = R.0
    polynomial_monomial_coeff_of_ne(Nat.3, r, Nat.2)
    Nat.2 != Nat.3
    Polynomial[R].monomial(Nat.3, r).coeff(Nat.2) = R.0
    coeff_tail(coeff_tail(Polynomial[R].monomial(Nat.3, r).coeff))(Nat.0) =
        Polynomial[R].monomial(Nat.3, r).coeff(Nat.2)
    coeff_eval(coeff_tail(coeff_tail(Polynomial[R].monomial(Nat.3, r).coeff)), x, Nat.0.suc.suc) =
        Polynomial[R].monomial(Nat.3, r).coeff(Nat.2) +
        x * coeff_eval(coeff_tail(coeff_tail(coeff_tail(
            Polynomial[R].monomial(Nat.3, r).coeff))), x, Nat.0.suc)
    coeff_tail(coeff_tail(coeff_tail(Polynomial[R].monomial(Nat.3, r).coeff)))(Nat.0) =
        coeff_tail(coeff_tail(Polynomial[R].monomial(Nat.3, r).coeff))(Nat.0.suc)
    coeff_tail(coeff_tail(Polynomial[R].monomial(Nat.3, r).coeff))(Nat.0.suc) =
        coeff_tail(Polynomial[R].monomial(Nat.3, r).coeff)(Nat.0.suc.suc)
    coeff_tail(Polynomial[R].monomial(Nat.3, r).coeff)(Nat.0.suc.suc) =
        Polynomial[R].monomial(Nat.3, r).coeff(Nat.0.suc.suc.suc)
    coeff_tail(Polynomial[R].monomial(Nat.3, r).coeff)(Nat.0.suc.suc) =
        Polynomial[R].monomial(Nat.3, r).coeff(Nat.3)
    coeff_tail(coeff_tail(coeff_tail(Polynomial[R].monomial(Nat.3, r).coeff)))(Nat.0) =
        Polynomial[R].monomial(Nat.3, r).coeff(Nat.3)
    coeff_eval(coeff_tail(coeff_tail(coeff_tail(Polynomial[R].monomial(Nat.3, r).coeff))), x,
        Nat.0.suc) =
        Polynomial[R].monomial(Nat.3, r).coeff(Nat.3) +
        x * coeff_eval(coeff_tail(coeff_tail(coeff_tail(coeff_tail(
            Polynomial[R].monomial(Nat.3, r).coeff)))), x, Nat.0)
    polynomial_monomial_coeff_self(Nat.3, r)
    Polynomial[R].monomial(Nat.3, r).coeff(Nat.3) = r
    coeff_eval_nat_zero(coeff_tail(coeff_tail(coeff_tail(coeff_tail(
        Polynomial[R].monomial(Nat.3, r).coeff)))), x)
    coeff_eval(coeff_tail(coeff_tail(coeff_tail(coeff_tail(
        Polynomial[R].monomial(Nat.3, r).coeff)))), x, Nat.0) = R.0
    coeff_eval(Polynomial[R].monomial(Nat.3, r).coeff, x, Nat.0.suc.suc.suc.suc) =
        R.0 + x * (R.0 + x * (R.0 + x * (r + x * R.0)))
    R.0 + x * (R.0 + x * (R.0 + x * (r + x * R.0))) = r * x * x * x
    polynomial_eval_bound(Polynomial[R].monomial(Nat.3, r), x, Nat.0.suc.suc.suc.suc) = r * x * x * x
    polynomial_eval(Polynomial[R].monomial(Nat.3, r), x) = r * x * x * x
}

/// Evaluation of a cubic polynomial at a point.
theorem cubic_polynomial_eval[R: CommRing](a: R, b: R, c: R, d: R, x: R) {
    polynomial_eval(cubic_polynomial(a, b, c, d), x) = d + c * x + b * x * x + a * x * x * x
} by {
    polynomial_eval(cubic_polynomial(a, b, c, d), x) =
        polynomial_eval((polynomial_constant(d) + polynomial_monomial(Nat.1, c) +
            polynomial_monomial(Nat.2, b)) + polynomial_monomial(Nat.3, a), x)
    polynomial_eval_add(polynomial_constant(d) + polynomial_monomial(Nat.1, c) +
        polynomial_monomial(Nat.2, b), polynomial_monomial(Nat.3, a), x)
    polynomial_eval((polynomial_constant(d) + polynomial_monomial(Nat.1, c) +
        polynomial_monomial(Nat.2, b)) + polynomial_monomial(Nat.3, a), x) =
        polynomial_eval(polynomial_constant(d) + polynomial_monomial(Nat.1, c) +
            polynomial_monomial(Nat.2, b), x) + polynomial_eval(polynomial_monomial(Nat.3, a), x)
    polynomial_eval_add(polynomial_constant(d) + polynomial_monomial(Nat.1, c),
        polynomial_monomial(Nat.2, b), x)
    polynomial_eval(polynomial_constant(d) + polynomial_monomial(Nat.1, c) +
        polynomial_monomial(Nat.2, b), x) =
        polynomial_eval(polynomial_constant(d) + polynomial_monomial(Nat.1, c), x) +
        polynomial_eval(polynomial_monomial(Nat.2, b), x)
    polynomial_eval_add(polynomial_constant(d), polynomial_monomial(Nat.1, c), x)
    polynomial_eval(polynomial_constant(d) + polynomial_monomial(Nat.1, c), x) =
        polynomial_eval(polynomial_constant(d), x) + polynomial_eval(polynomial_monomial(Nat.1, c), x)
    polynomial_eval_constant(d, x)
    polynomial_eval(polynomial_constant(d), x) = d
    monomial_one_eval(c, x)
    polynomial_eval(polynomial_monomial(Nat.1, c), x) = c * x
    monomial_two_eval(b, x)
    polynomial_eval(polynomial_monomial(Nat.2, b), x) = b * x * x
    monomial_three_eval(a, x)
    polynomial_eval(polynomial_monomial(Nat.3, a), x) = a * x * x * x
    polynomial_eval(cubic_polynomial(a, b, c, d), x) = d + c * x + b * x * x + a * x * x * x
}

// ---------------------------------------------------------------------------
// 3b. Concrete integer arithmetic for the cubic `X^3 - 6 X^2 + 11 X - 6`
// ---------------------------------------------------------------------------

/// `-from_nat(m) + from_nat(n) = sub_nat(n, m)`.
theorem int_neg_from_nat_add(n: Nat, m: Nat) {
    -Int.from_nat(m) + Int.from_nat(n) = sub_nat(n, m)
} by {
    sub_nat_zero_left(m)
    -Int.from_nat(m) = sub_nat(Nat.0, m)
    sub_nat_zero_right(n)
    Int.from_nat(n) = sub_nat(n, Nat.0)
    add_sub_nat(Nat.0, m, n, Nat.0)
    sub_nat(Nat.0, m) + sub_nat(n, Nat.0) = sub_nat(Nat.0 + n, m + Nat.0)
    sub_nat(Nat.0 + n, m + Nat.0) = sub_nat(n, m)
    -Int.from_nat(m) + Int.from_nat(n) = sub_nat(n, m)
}

/// `-from_nat(m) + from_nat(n) = from_nat(k)` when `m + k = n`.
theorem int_neg_from_nat_add_cancel(m: Nat, n: Nat, k: Nat) {
    m + k = n implies -Int.from_nat(m) + Int.from_nat(n) = Int.from_nat(k)
} by {
    if m + k = n {
        int_neg_from_nat_add(n, m)
        -Int.from_nat(m) + Int.from_nat(n) = sub_nat(n, m)
        sub_nat_add_left(k, m)
        sub_nat(m + k, m) = Int.from_nat(k)
        m + k = n
        sub_nat(n, m) = Int.from_nat(k)
        -Int.from_nat(m) + Int.from_nat(n) = Int.from_nat(k)
    }
}

/// `-from_nat(m) + from_nat(n) = -from_nat(k)` when `n + k = m`.
theorem int_neg_from_nat_add_neg(m: Nat, n: Nat, k: Nat) {
    n + k = m implies -Int.from_nat(m) + Int.from_nat(n) = -Int.from_nat(k)
} by {
    if n + k = m {
        int_neg_from_nat_add(n, m)
        -Int.from_nat(m) + Int.from_nat(n) = sub_nat(n, m)
        neg_sub_nat(n, m)
        sub_nat(n, m) = -(sub_nat(m, n))
        sub_nat_add_left(k, n)
        sub_nat(n + k, n) = Int.from_nat(k)
        n + k = m
        sub_nat(m, n) = Int.from_nat(k)
        -(sub_nat(m, n)) = -Int.from_nat(k)
        sub_nat(n, m) = -Int.from_nat(k)
        -Int.from_nat(m) + Int.from_nat(n) = -Int.from_nat(k)
    }
}

/// `from_nat(m) * -from_nat(n) = -from_nat(m * n)`.
theorem int_mul_neg_from_nat(m: Nat, n: Nat) {
    Int.from_nat(m) * -Int.from_nat(n) = -Int.from_nat(m * n)
} by {
    mul_neg_right(Int.from_nat(m), Int.from_nat(n))
    Int.from_nat(m) * -Int.from_nat(n) = -(Int.from_nat(m) * Int.from_nat(n))
    mul_from_nat(m, n)
    Int.from_nat(m) * Int.from_nat(n) = Int.from_nat(m * n)
    -(Int.from_nat(m) * Int.from_nat(n)) = -Int.from_nat(m * n)
    Int.from_nat(m) * -Int.from_nat(n) = -Int.from_nat(m * n)
}

/// `-6 + 2 = -4`.
theorem int_neg_six_add_two {
    -Int.6 + Int.2 = -Int.4
} by {
    int_neg_from_nat_add_neg(Nat.6, Nat.2, Nat.4)
    Nat.2 + Nat.4 = Nat.6
    -Int.from_nat(Nat.6) + Int.from_nat(Nat.2) = -Int.from_nat(Nat.4)
    Int.6 = Int.from_nat(Nat.6)
    Int.2 = Int.from_nat(Nat.2)
    Int.4 = Int.from_nat(Nat.4)
    -Int.6 + Int.2 = -Int.4
}

/// `11 + -8 = 3`.
theorem int_eleven_add_neg_eight {
    Int.11 + -Int.8 = Int.3
} by {
    int_neg_from_nat_add_cancel(Nat.8, Nat.11, Nat.3)
    Nat.3 = Nat.2.suc
    Nat.8 + Nat.3 = Nat.8 + Nat.2.suc
    add_suc_right(Nat.8, Nat.2)
    Nat.8 + Nat.2.suc = (Nat.8 + Nat.2).suc
    Nat.8 + Nat.2 = Nat.10
    (Nat.8 + Nat.2).suc = Nat.10.suc
    Nat.10.suc = Nat.11
    Nat.8 + Nat.3 = Nat.11
    -Int.from_nat(Nat.8) + Int.from_nat(Nat.11) = Int.from_nat(Nat.3)
    Int.11 = Int.from_nat(Nat.11)
    Int.8 = Int.from_nat(Nat.8)
    Int.3 = Int.from_nat(Nat.3)
    Int.11 + -Int.8 = Int.3
}

/// `2 * -4 = -8`.
theorem int_two_mul_neg_four {
    Int.2 * -Int.4 = -Int.8
} by {
    mul_neg_right(Int.2, Int.4)
    Int.2 * -Int.4 = -(Int.2 * Int.4)
    mul_from_nat(Nat.2, Nat.4)
    Int.2 * Int.4 = Int.from_nat(Nat.2 * Nat.4)
    Nat.2 * Nat.4 = Nat.8
    Int.from_nat(Nat.2 * Nat.4) = Int.from_nat(Nat.8)
    Int.from_nat(Nat.8) = Int.8
    Int.2 * Int.4 = Int.8
    -(Int.2 * Int.4) = -Int.8
    Int.2 * -Int.4 = -Int.8
}

/// `2 * 3 = 6`.
theorem int_two_mul_three {
    Int.2 * Int.3 = Int.6
} by {
    mul_from_nat(Nat.2, Nat.3)
    Int.2 * Int.3 = Int.from_nat(Nat.2 * Nat.3)
    Nat.2 * Nat.3 = Nat.6
    Int.from_nat(Nat.2 * Nat.3) = Int.from_nat(Nat.6)
    Int.from_nat(Nat.6) = Int.6
    Int.2 * Int.3 = Int.6
}

/// `-6 + 3 = -3`.
theorem int_neg_six_add_three {
    -Int.6 + Int.3 = -Int.3
} by {
    int_neg_from_nat_add_neg(Nat.6, Nat.3, Nat.3)
    Nat.3 + Nat.3 = Nat.6
    -Int.from_nat(Nat.6) + Int.from_nat(Nat.3) = -Int.from_nat(Nat.3)
    Int.6 = Int.from_nat(Nat.6)
    Int.3 = Int.from_nat(Nat.3)
    -Int.6 + Int.3 = -Int.3
}

/// `11 + -9 = 2`.
theorem int_eleven_add_neg_nine {
    Int.11 + -Int.9 = Int.2
} by {
    int_neg_from_nat_add_cancel(Nat.9, Nat.11, Nat.2)
    Nat.2 = Nat.1.suc
    Nat.9 + Nat.2 = Nat.9 + Nat.1.suc
    add_suc_right(Nat.9, Nat.1)
    Nat.9 + Nat.1.suc = (Nat.9 + Nat.1).suc
    Nat.9 + Nat.1 = Nat.10
    (Nat.9 + Nat.1).suc = Nat.10.suc
    Nat.10.suc = Nat.11
    Nat.9 + Nat.2 = Nat.11
    -Int.from_nat(Nat.9) + Int.from_nat(Nat.11) = Int.from_nat(Nat.2)
    Int.11 = Int.from_nat(Nat.11)
    Int.9 = Int.from_nat(Nat.9)
    Int.2 = Int.from_nat(Nat.2)
    Int.11 + -Int.9 = Int.2
}

/// `3 * -3 = -9`.
theorem int_three_mul_neg_three {
    Int.3 * -Int.3 = -Int.9
} by {
    mul_neg_right(Int.3, Int.3)
    Int.3 * -Int.3 = -(Int.3 * Int.3)
    mul_from_nat(Nat.3, Nat.3)
    Int.3 * Int.3 = Int.from_nat(Nat.3 * Nat.3)
    Nat.3 * Nat.3 = Nat.9
    Int.from_nat(Nat.3 * Nat.3) = Int.from_nat(Nat.9)
    Int.from_nat(Nat.9) = Int.9
    Int.3 * Int.3 = Int.9
    -(Int.3 * Int.3) = -Int.9
    Int.3 * -Int.3 = -Int.9
}

/// `3 * 2 = 6`.
theorem int_three_mul_two {
    Int.3 * Int.2 = Int.6
} by {
    mul_from_nat(Nat.3, Nat.2)
    Int.3 * Int.2 = Int.from_nat(Nat.3 * Nat.2)
    Nat.3 * Nat.2 = Nat.6
    Int.from_nat(Nat.3 * Nat.2) = Int.from_nat(Nat.6)
    Int.from_nat(Nat.6) = Int.6
    Int.3 * Int.2 = Int.6
}

/// The cubic `X^3 - 6 X^2 + 11 X - 6` with integer coefficients.
let int_cubic_x3_minus_6x2_plus_11x_minus_6: Polynomial[Int] =
    cubic_polynomial[Int](Int.1, -Int.6, Int.11, -Int.6)

/// The integer cubic `X^3 - 6 X^2 + 11 X - 6` has the integer root `1`.
theorem int_cubic_x3_minus_6x2_plus_11x_minus_6_root_one {
    polynomial_eval(int_cubic_x3_minus_6x2_plus_11x_minus_6, Int.1) = Int.0
} by {
    cubic_polynomial_eval(Int.1, -Int.6, Int.11, -Int.6, Int.1)
    polynomial_eval(int_cubic_x3_minus_6x2_plus_11x_minus_6, Int.1) =
        -Int.6 + Int.11 * Int.1 + (-Int.6) * Int.1 * Int.1 + Int.1 * Int.1 * Int.1 * Int.1
    -Int.6 + Int.11 * Int.1 + (-Int.6) * Int.1 * Int.1 + Int.1 * Int.1 * Int.1 * Int.1 =
        -Int.6 + Int.11 + -Int.6 + Int.1
    int_neg_from_nat_add_cancel(Nat.6, Nat.11, Nat.5)
    Nat.5 = Nat.4.suc
    Nat.6 + Nat.5 = Nat.6 + Nat.4.suc
    add_suc_right(Nat.6, Nat.4)
    Nat.6 + Nat.4.suc = (Nat.6 + Nat.4).suc
    Nat.6 + Nat.4 = Nat.10
    (Nat.6 + Nat.4).suc = Nat.10.suc
    Nat.10.suc = Nat.11
    Nat.6 + Nat.5 = Nat.11
    -Int.from_nat(Nat.6) + Int.from_nat(Nat.11) = Int.from_nat(Nat.5)
    Int.6 = Int.from_nat(Nat.6)
    Int.11 = Int.from_nat(Nat.11)
    Int.5 = Int.from_nat(Nat.5)
    -Int.6 + Int.11 = Int.5
    -Int.6 + Int.11 + -Int.6 + Int.1 = Int.5 + -Int.6 + Int.1
    int_neg_from_nat_add_neg(Nat.6, Nat.5, Nat.1)
    Nat.5 + Nat.1 = Nat.6
    -Int.from_nat(Nat.6) + Int.from_nat(Nat.5) = -Int.from_nat(Nat.1)
    Int.1 = Int.from_nat(Nat.1)
    Int.5 + -Int.6 = -Int.1
    Int.5 + -Int.6 + Int.1 = -Int.1 + Int.1
    add_neg(Int.1)
    Int.1 + -Int.1 = Int.0
    -Int.1 + Int.1 = Int.0
    Int.5 + -Int.6 + Int.1 = Int.0
    -Int.6 + Int.11 + -Int.6 + Int.1 = Int.0
    polynomial_eval(int_cubic_x3_minus_6x2_plus_11x_minus_6, Int.1) = Int.0
}

/// The integer cubic `X^3 - 6 X^2 + 11 X - 6` has the integer root `2`.
///
/// The value is computed in the Horner form at bound four, where every
/// intermediate product and sum stays small.
theorem int_cubic_x3_minus_6x2_plus_11x_minus_6_root_two {
    polynomial_eval(int_cubic_x3_minus_6x2_plus_11x_minus_6, Int.2) = Int.0
} by {
    cubic_polynomial_support_bounded_by_four(Int.1, -Int.6, Int.11, -Int.6)
    polynomial_support_bounded_by(int_cubic_x3_minus_6x2_plus_11x_minus_6, Nat.4)
    polynomial_eval_eq_eval_bound_of_support_bounded(int_cubic_x3_minus_6x2_plus_11x_minus_6, Int.2, Nat.4)
    polynomial_eval(int_cubic_x3_minus_6x2_plus_11x_minus_6, Int.2) =
        polynomial_eval_bound(int_cubic_x3_minus_6x2_plus_11x_minus_6, Int.2, Nat.4)
    polynomial_eval_bound_eq_coeff_eval(int_cubic_x3_minus_6x2_plus_11x_minus_6, Int.2, Nat.4)
    polynomial_eval_bound(int_cubic_x3_minus_6x2_plus_11x_minus_6, Int.2, Nat.4) =
        coeff_eval(int_cubic_x3_minus_6x2_plus_11x_minus_6.coeff, Int.2, Nat.4)
    coeff_eval_four(int_cubic_x3_minus_6x2_plus_11x_minus_6.coeff, Int.2)
    coeff_eval(int_cubic_x3_minus_6x2_plus_11x_minus_6.coeff, Int.2, Nat.4) =
        int_cubic_x3_minus_6x2_plus_11x_minus_6.coeff(Nat.0) +
        Int.2 * (int_cubic_x3_minus_6x2_plus_11x_minus_6.coeff(Nat.1) +
            Int.2 * (int_cubic_x3_minus_6x2_plus_11x_minus_6.coeff(Nat.2) +
                Int.2 * (int_cubic_x3_minus_6x2_plus_11x_minus_6.coeff(Nat.3) + Int.2 * Int.0)))
    cubic_polynomial_coeff_zero(Int.1, -Int.6, Int.11, -Int.6)
    int_cubic_x3_minus_6x2_plus_11x_minus_6.coeff(Nat.0) = -Int.6
    cubic_polynomial_coeff_one(Int.1, -Int.6, Int.11, -Int.6)
    int_cubic_x3_minus_6x2_plus_11x_minus_6.coeff(Nat.1) = Int.11
    cubic_polynomial_coeff_two(Int.1, -Int.6, Int.11, -Int.6)
    int_cubic_x3_minus_6x2_plus_11x_minus_6.coeff(Nat.2) = -Int.6
    cubic_polynomial_coeff_three(Int.1, -Int.6, Int.11, -Int.6)
    int_cubic_x3_minus_6x2_plus_11x_minus_6.coeff(Nat.3) = Int.1
    coeff_eval(int_cubic_x3_minus_6x2_plus_11x_minus_6.coeff, Int.2, Nat.4) =
        -Int.6 + Int.2 * (Int.11 + Int.2 * (-Int.6 + Int.2 * (Int.1 + Int.2 * Int.0)))
    -Int.6 + Int.2 * (Int.11 + Int.2 * (-Int.6 + Int.2 * (Int.1 + Int.2 * Int.0))) =
        -Int.6 + Int.2 * (Int.11 + Int.2 * (-Int.6 + Int.2 * Int.1))
    int_neg_six_add_two
    -Int.6 + Int.2 = -Int.4
    -Int.6 + Int.2 * (Int.11 + Int.2 * (-Int.6 + Int.2 * Int.1)) =
        -Int.6 + Int.2 * (Int.11 + Int.2 * -Int.4)
    int_two_mul_neg_four
    Int.2 * -Int.4 = -Int.8
    -Int.6 + Int.2 * (Int.11 + Int.2 * -Int.4) = -Int.6 + Int.2 * (Int.11 + -Int.8)
    int_eleven_add_neg_eight
    Int.11 + -Int.8 = Int.3
    -Int.6 + Int.2 * (Int.11 + -Int.8) = -Int.6 + Int.2 * Int.3
    int_two_mul_three
    Int.2 * Int.3 = Int.6
    -Int.6 + Int.2 * Int.3 = -Int.6 + Int.6
    add_neg(Int.6)
    Int.6 + -Int.6 = Int.0
    -Int.6 + Int.6 = Int.0
    -Int.6 + Int.2 * Int.3 = Int.0
    coeff_eval(int_cubic_x3_minus_6x2_plus_11x_minus_6.coeff, Int.2, Nat.4) = Int.0
    polynomial_eval(int_cubic_x3_minus_6x2_plus_11x_minus_6, Int.2) = Int.0
}

/// The integer cubic `X^3 - 6 X^2 + 11 X - 6` has the integer root `3`.
///
/// The value is computed in the Horner form at bound four, where every
/// intermediate product and sum stays small.
theorem int_cubic_x3_minus_6x2_plus_11x_minus_6_root_three {
    polynomial_eval(int_cubic_x3_minus_6x2_plus_11x_minus_6, Int.3) = Int.0
} by {
    cubic_polynomial_support_bounded_by_four(Int.1, -Int.6, Int.11, -Int.6)
    polynomial_support_bounded_by(int_cubic_x3_minus_6x2_plus_11x_minus_6, Nat.4)
    polynomial_eval_eq_eval_bound_of_support_bounded(int_cubic_x3_minus_6x2_plus_11x_minus_6, Int.3, Nat.4)
    polynomial_eval(int_cubic_x3_minus_6x2_plus_11x_minus_6, Int.3) =
        polynomial_eval_bound(int_cubic_x3_minus_6x2_plus_11x_minus_6, Int.3, Nat.4)
    polynomial_eval_bound_eq_coeff_eval(int_cubic_x3_minus_6x2_plus_11x_minus_6, Int.3, Nat.4)
    polynomial_eval_bound(int_cubic_x3_minus_6x2_plus_11x_minus_6, Int.3, Nat.4) =
        coeff_eval(int_cubic_x3_minus_6x2_plus_11x_minus_6.coeff, Int.3, Nat.4)
    coeff_eval_four(int_cubic_x3_minus_6x2_plus_11x_minus_6.coeff, Int.3)
    coeff_eval(int_cubic_x3_minus_6x2_plus_11x_minus_6.coeff, Int.3, Nat.4) =
        int_cubic_x3_minus_6x2_plus_11x_minus_6.coeff(Nat.0) +
        Int.3 * (int_cubic_x3_minus_6x2_plus_11x_minus_6.coeff(Nat.1) +
            Int.3 * (int_cubic_x3_minus_6x2_plus_11x_minus_6.coeff(Nat.2) +
                Int.3 * (int_cubic_x3_minus_6x2_plus_11x_minus_6.coeff(Nat.3) + Int.3 * Int.0)))
    cubic_polynomial_coeff_zero(Int.1, -Int.6, Int.11, -Int.6)
    int_cubic_x3_minus_6x2_plus_11x_minus_6.coeff(Nat.0) = -Int.6
    cubic_polynomial_coeff_one(Int.1, -Int.6, Int.11, -Int.6)
    int_cubic_x3_minus_6x2_plus_11x_minus_6.coeff(Nat.1) = Int.11
    cubic_polynomial_coeff_two(Int.1, -Int.6, Int.11, -Int.6)
    int_cubic_x3_minus_6x2_plus_11x_minus_6.coeff(Nat.2) = -Int.6
    cubic_polynomial_coeff_three(Int.1, -Int.6, Int.11, -Int.6)
    int_cubic_x3_minus_6x2_plus_11x_minus_6.coeff(Nat.3) = Int.1
    coeff_eval(int_cubic_x3_minus_6x2_plus_11x_minus_6.coeff, Int.3, Nat.4) =
        -Int.6 + Int.3 * (Int.11 + Int.3 * (-Int.6 + Int.3 * (Int.1 + Int.3 * Int.0)))
    -Int.6 + Int.3 * (Int.11 + Int.3 * (-Int.6 + Int.3 * (Int.1 + Int.3 * Int.0))) =
        -Int.6 + Int.3 * (Int.11 + Int.3 * (-Int.6 + Int.3 * Int.1))
    int_neg_six_add_three
    -Int.6 + Int.3 = -Int.3
    -Int.6 + Int.3 * (Int.11 + Int.3 * (-Int.6 + Int.3 * Int.1)) =
        -Int.6 + Int.3 * (Int.11 + Int.3 * -Int.3)
    int_three_mul_neg_three
    Int.3 * -Int.3 = -Int.9
    -Int.6 + Int.3 * (Int.11 + Int.3 * -Int.3) = -Int.6 + Int.3 * (Int.11 + -Int.9)
    int_eleven_add_neg_nine
    Int.11 + -Int.9 = Int.2
    -Int.6 + Int.3 * (Int.11 + -Int.9) = -Int.6 + Int.3 * Int.2
    int_three_mul_two
    Int.3 * Int.2 = Int.6
    -Int.6 + Int.3 * Int.2 = -Int.6 + Int.6
    add_neg(Int.6)
    Int.6 + -Int.6 = Int.0
    -Int.6 + Int.6 = Int.0
    -Int.6 + Int.3 * Int.2 = Int.0
    coeff_eval(int_cubic_x3_minus_6x2_plus_11x_minus_6.coeff, Int.3, Nat.4) = Int.0
    polynomial_eval(int_cubic_x3_minus_6x2_plus_11x_minus_6, Int.3) = Int.0
}

// ---------------------------------------------------------------------------
// 4. The fundamental theorem of algebra for cubics
// ---------------------------------------------------------------------------

// The fundamental theorem of algebra for cubics: every real cubic
// `a X^3 + b X^2 + c X + d` with nonzero leading coefficient has a real root.
//
//     theorem real_cubic_has_a_real_root(a: Real, b: Real, c: Real, d: Real) {
//         a != Real.0 implies exists(x: Real) {
//             polynomial_eval(cubic_polynomial[Real](a, b, c, d), x) = Real.0
//         }
//     }
//
// By the sign-change root theorem `real_polynomial_sign_change_has_root`
// (Bolzano's theorem, i.e. the intermediate value theorem at target zero, in
// `src/number_theory/polynomial_nt.ac`), it suffices to show the cubic takes
// both signs somewhere.  That is the leading-term dominance lemma for degree
// three, which needs real limit machinery not yet in the library (see the
// odd-degree discussion in `polynomial_nt.ac`), so the general statement is
// recorded here only.

/// The cubic `X^3 - 1` with real coefficients.
let real_x_cubed_minus_one: Polynomial[Real] = cubic_polynomial[Real](Real.1, Real.0, Real.0, -Real.1)

/// The real cubic `X^3 - 1` has the real root `1`.
theorem real_x_cubed_minus_one_root_one {
    polynomial_eval(real_x_cubed_minus_one, Real.1) = Real.0
} by {
    cubic_polynomial_eval(Real.1, Real.0, Real.0, -Real.1, Real.1)
    polynomial_eval(real_x_cubed_minus_one, Real.1) =
        -Real.1 + Real.0 * Real.1 + Real.0 * Real.1 * Real.1 + Real.1 * Real.1 * Real.1 * Real.1
    Real.0 * Real.1 = Real.0
    Real.1 * Real.1 * Real.1 * Real.1 = Real.1
    -Real.1 + Real.0 * Real.1 + Real.0 * Real.1 * Real.1 + Real.1 * Real.1 * Real.1 * Real.1 =
        -Real.1 + Real.0 + Real.0 + Real.1
    -Real.1 + Real.0 + Real.0 + Real.1 = -Real.1 + Real.1
    -Real.1 + Real.1 = Real.0
    polynomial_eval(real_x_cubed_minus_one, Real.1) = Real.0
}

// ---------------------------------------------------------------------------
// 3d. The cubic `X^3 - X`: three roots with trivial arithmetic
// ---------------------------------------------------------------------------

/// The cubic `X^3 - X` with integer coefficients.
let int_cubic_x3_minus_x: Polynomial[Int] = cubic_polynomial[Int](Int.1, Int.0, -Int.1, Int.0)

/// `X^3 - X` has the integer root `0`.
theorem int_cubic_x3_minus_x_root_zero {
    polynomial_eval(int_cubic_x3_minus_x, Int.0) = Int.0
} by {
    cubic_polynomial_eval(Int.1, Int.0, -Int.1, Int.0, Int.0)
    polynomial_eval(int_cubic_x3_minus_x, Int.0) =
        Int.0 + (-Int.1) * Int.0 + Int.0 * Int.0 * Int.0 + Int.1 * Int.0 * Int.0 * Int.0
    (-Int.1) * Int.0 = Int.0
    Int.0 + (-Int.1) * Int.0 + Int.0 * Int.0 * Int.0 + Int.1 * Int.0 * Int.0 * Int.0 = Int.0
    polynomial_eval(int_cubic_x3_minus_x, Int.0) = Int.0
}

/// `X^3 - X` has the integer root `1`.
theorem int_cubic_x3_minus_x_root_one {
    polynomial_eval(int_cubic_x3_minus_x, Int.1) = Int.0
} by {
    cubic_polynomial_eval(Int.1, Int.0, -Int.1, Int.0, Int.1)
    polynomial_eval(int_cubic_x3_minus_x, Int.1) =
        Int.0 + (-Int.1) * Int.1 + Int.0 * Int.1 * Int.1 + Int.1 * Int.1 * Int.1 * Int.1
    Int.0 + (-Int.1) * Int.1 + Int.0 * Int.1 * Int.1 + Int.1 * Int.1 * Int.1 * Int.1 =
        -Int.1 + Int.1
    add_neg(Int.1)
    Int.1 + -Int.1 = Int.0
    -Int.1 + Int.1 = Int.0
    polynomial_eval(int_cubic_x3_minus_x, Int.1) = Int.0
}

/// `X^3 - X` has the integer root `-1`.
theorem int_cubic_x3_minus_x_root_neg_one {
    polynomial_eval(int_cubic_x3_minus_x, -Int.1) = Int.0
} by {
    cubic_polynomial_eval(Int.1, Int.0, -Int.1, Int.0, -Int.1)
    polynomial_eval(int_cubic_x3_minus_x, -Int.1) =
        Int.0 + (-Int.1) * (-Int.1) + Int.0 * (-Int.1) * (-Int.1) +
            Int.1 * (-Int.1) * (-Int.1) * (-Int.1)
    mul_neg_neg(Int.1, Int.1)
    (-Int.1) * (-Int.1) = Int.1 * Int.1
    Int.1 * Int.1 = Int.1
    (-Int.1) * (-Int.1) = Int.1
    Int.1 * (-Int.1) * (-Int.1) * (-Int.1) = Int.1 * (-Int.1) * (-Int.1) * (-Int.1)
    Int.1 * (-Int.1) * (-Int.1) * (-Int.1) = -Int.1
    Int.0 + (-Int.1) * (-Int.1) + Int.0 * (-Int.1) * (-Int.1) +
        Int.1 * (-Int.1) * (-Int.1) * (-Int.1) = Int.1 + -Int.1
    add_neg(Int.1)
    Int.1 + -Int.1 = Int.0
    Int.1 + -Int.1 = Int.0
    polynomial_eval(int_cubic_x3_minus_x, -Int.1) = Int.0
}


// ---------------------------------------------------------------------------
// 3c. The rational root theorem for cubics
// ---------------------------------------------------------------------------

// The rational root theorem for cubics is proved in the interface universe of
// the polynomial module, in `src/polynomial/polynomial_rational_root_cubic.ac`
// (the coefficient-level rational root theorem of
// `src/polynomial_rational_root_bridge.ac` is stated in terms of the interface
// `coeff_eval`, which a submodule file cannot import together with the
// implementation `coeff_eval`).  The concrete cubic `X^3 - 6 X^2 + 11 X - 6`,
// its rational roots `1`, `2`, `3`, and the divisibility `1 | 6`, `2 | 6`,
// `3 | 6` are verified there.

from polynomial.polynomial_rational_root_cubic import rat_cubic_polynomial,
    cubic_rational_root_numerator_divides,
    cubic_rational_root_denominator_divides, rat_cubic_x3_minus_6x2_plus_11x_minus_6_root_one,
    rat_cubic_x3_minus_6x2_plus_11x_minus_6_root_two, rat_cubic_x3_minus_6x2_plus_11x_minus_6_root_three,
    int_cubic_x3_minus_6x2_plus_11x_minus_6_root_one_divides_six,
    int_cubic_x3_minus_6x2_plus_11x_minus_6_root_two_divides_six,
    int_cubic_x3_minus_6x2_plus_11x_minus_6_root_three_divides_six

// ---------------------------------------------------------------------------
// 5. The power rule for derivatives (statement)
// ---------------------------------------------------------------------------

// The power rule for natural powers on the reals:
//
//     theorem derivative_power_rule(n: Nat, x: Real) {
//         has_derivative_at(pow_real_fn(n), x,
//             Real.from_nat(n) * x.pow(n - Nat.1))
//     }
//
// The proof is an induction over `n` using the product rule
// (`derivative_pointwise_mul`), `pow_real_fn_suc`, and the identity and
// constant derivative theorems of `src/real/`; the inductive step reduces to
// `n x^(n-1) * x + x^n = (n + 1) x^n`, which needs natural-number arithmetic
// on the embedded coefficients.  It is recorded here as a statement only; the
// concrete power derivatives for small `n` are verified in
// `src/real/derivative_affine_*`.
