/// Transport of polynomial coefficients along semiring homomorphisms.

from finite_set import FiniteSet
from algebra.module.finite_support import has_support_in, has_support_in_apply_zero, is_finitely_supported,
    is_finitely_supported_of_has_support_in
from data.basic.functions import function_extensionality
from nat import Nat, alt_induction
from polynomial.base import Polynomial
from polynomial.add_monoid_algebra_bridge import coeff_zero_at, coeff_zero_from,
    coeff_zero_from_at
from polynomial.coeff_eval import coeff_eval, coeff_tail
from polynomial.eval import polynomial_support_bounded_by, polynomial_eval_bound
from polynomial.global_eval import polynomial_eval, polynomial_eval_eq_eval_bound_of_support_bounded,
    polynomial_support_bound_exists
from semiring import Semiring
from algebra.semiring_hom import SemiringHom, semiring_hom_zero, semiring_hom_add, semiring_hom_mul

/// The coefficient function obtained by applying a semiring homomorphism.
define polynomial_map_coeff[R: Semiring, S: Semiring](
    f: SemiringHom[R, S],
    p: Polynomial[R],
    n: Nat
) -> S {
    f.hom(p.coeff(n))
}

/// Transported coefficients are finitely supported.
theorem polynomial_map_coeff_is_finitely_supported[R: Semiring, S: Semiring](
    f: SemiringHom[R, S],
    p: Polynomial[R]
) {
    is_finitely_supported(polynomial_map_coeff(f, p))
} by {
    is_finitely_supported(p.coeff)
    let support: FiniteSet[Nat] satisfy {
        has_support_in(p.coeff, support)
    }
    forall(n: Nat) {
        if not support.contains(n) {
            has_support_in_apply_zero(p.coeff, support, n)
            p.coeff(n) = R.0
            semiring_hom_zero(f)
            f.hom(R.0) = S.0
            polynomial_map_coeff(f, p, n) = f.hom(p.coeff(n))
            polynomial_map_coeff(f, p, n) = S.0
        }
    }
    has_support_in(polynomial_map_coeff(f, p), support)
    is_finitely_supported_of_has_support_in(polynomial_map_coeff(f, p), support)
}

/// The polynomial obtained by applying a semiring homomorphism to every coefficient.
let polynomial_map[R: Semiring, S: Semiring](f: SemiringHom[R, S], p: Polynomial[R]) -> result: Polynomial[S] satisfy {
    Polynomial[S].new(polynomial_map_coeff(f, p)) = Option.some(result)
} by {
    polynomial_map_coeff_is_finitely_supported(f, p)
}

/// A transported polynomial has the transported coefficients.
theorem polynomial_map_coeff_apply[R: Semiring, S: Semiring](
    f: SemiringHom[R, S],
    p: Polynomial[R],
    n: Nat
) {
    polynomial_map(f, p).coeff(n) = f.hom(p.coeff(n))
} by {
    Polynomial[S].new(polynomial_map_coeff(f, p)) = Option.some(polynomial_map(f, p))
    polynomial_map(f, p).coeff = polynomial_map_coeff(f, p)
    polynomial_map(f, p).coeff(n) = polynomial_map_coeff(f, p, n)
    polynomial_map_coeff(f, p, n) = f.hom(p.coeff(n))
}

/// The coefficient function obtained by applying a semiring homomorphism.
define polynomial_map_coeff_fn[R: Semiring, S: Semiring](
    f: SemiringHom[R, S],
    c: Nat -> R,
    n: Nat
) -> S {
    f.hom(c(n))
}

lemma polynomial_map_coeff_tail[R: Semiring, S: Semiring](
    f: SemiringHom[R, S],
    c: Nat -> R
) {
    coeff_tail(polynomial_map_coeff_fn(f, c)) = polynomial_map_coeff_fn(f, coeff_tail(c))
} by {
    forall(n: Nat) {
        coeff_tail(polynomial_map_coeff_fn(f, c), n) =
            polynomial_map_coeff_fn(f, c, n.suc)
        polynomial_map_coeff_fn(f, c, n.suc) = f.hom(c(n.suc))
        coeff_tail(c, n) = c(n.suc)
        polynomial_map_coeff_fn(f, coeff_tail(c), n) = f.hom(coeff_tail(c, n))
        coeff_tail(polynomial_map_coeff_fn(f, c), n) =
            polynomial_map_coeff_fn(f, coeff_tail(c), n)
    }
    function_extensionality(
        coeff_tail(polynomial_map_coeff_fn(f, c)),
        polynomial_map_coeff_fn(f, coeff_tail(c)))
}

/// Bounded Horner evaluation commutes with a semiring homomorphism.
theorem coeff_eval_map[R: Semiring, S: Semiring](
    f: SemiringHom[R, S],
    c: Nat -> R,
    x: R,
    n: Nat
) {
    coeff_eval(polynomial_map_coeff_fn(f, c), f.hom(x), n) = f.hom(coeff_eval(c, x, n))
} by {
    define statement(k: Nat) -> Bool {
        forall(d: Nat -> R, y: R) {
            coeff_eval(polynomial_map_coeff_fn(f, d), f.hom(y), k) =
                f.hom(coeff_eval(d, y, k))
        }
    }

    forall(d: Nat -> R, y: R) {
        coeff_eval(polynomial_map_coeff_fn(f, d), f.hom(y), Nat.0) = S.0
        coeff_eval(d, y, Nat.0) = R.0
        semiring_hom_zero(f)
        f.hom(coeff_eval(d, y, Nat.0)) = S.0
        coeff_eval(polynomial_map_coeff_fn(f, d), f.hom(y), Nat.0) =
            f.hom(coeff_eval(d, y, Nat.0))
    }
    statement(Nat.0)

    forall(k: Nat) {
        if statement(k) {
            forall(d: Nat -> R, y: R) {
                polynomial_map_coeff_tail(f, d)
                statement(k) = forall(e: Nat -> R, z: R) {
                    coeff_eval(polynomial_map_coeff_fn(f, e), f.hom(z), k) =
                        f.hom(coeff_eval(e, z, k))
                }
                coeff_eval(polynomial_map_coeff_fn(f, coeff_tail(d)), f.hom(y), k) =
                    f.hom(coeff_eval(coeff_tail(d), y, k))
                coeff_eval(polynomial_map_coeff_fn(f, d), f.hom(y), k.suc) =
                    f.hom(d(Nat.0)) + f.hom(y) *
                        coeff_eval(coeff_tail(polynomial_map_coeff_fn(f, d)), f.hom(y), k)
                coeff_eval(polynomial_map_coeff_fn(f, d), f.hom(y), k.suc) =
                    f.hom(d(Nat.0)) + f.hom(y) *
                        f.hom(coeff_eval(coeff_tail(d), y, k))
                semiring_hom_mul(f, y, coeff_eval(coeff_tail(d), y, k))
                semiring_hom_add(f, d(Nat.0), y * coeff_eval(coeff_tail(d), y, k))
                coeff_eval(d, y, k.suc) =
                    d(Nat.0) + y * coeff_eval(coeff_tail(d), y, k)
                coeff_eval(polynomial_map_coeff_fn(f, d), f.hom(y), k.suc) =
                    f.hom(coeff_eval(d, y, k.suc))
            }
            statement(k.suc)
        }
    }
    statement(Nat.0) and forall(k: Nat) {
        statement(k) implies statement(k.suc)
    }
    alt_induction(statement)
    forall(k: Nat) { statement(k) }
    statement(n)
    statement(n) = forall(d: Nat -> R, y: R) {
        coeff_eval(polynomial_map_coeff_fn(f, d), f.hom(y), n) =
            f.hom(coeff_eval(d, y, n))
    }
    coeff_eval(polynomial_map_coeff_fn(f, c), f.hom(x), n) =
        f.hom(coeff_eval(c, x, n))
}

/// A support bound remains valid after transporting coefficients.
theorem polynomial_map_support_bounded_by[R: Semiring, S: Semiring](
    f: SemiringHom[R, S],
    p: Polynomial[R],
    n: Nat
) {
    polynomial_support_bounded_by(p, n) implies
        polynomial_support_bounded_by(polynomial_map(f, p), n)
} by {
    if polynomial_support_bounded_by(p, n) {
        polynomial_support_bounded_by(p, n) = coeff_zero_from(p.coeff, n)
        coeff_zero_from(p.coeff, n) = forall(k: Nat) {
            coeff_zero_from_at(p.coeff, n, k)
        }
        forall(k: Nat) {
            coeff_zero_from_at(p.coeff, n, k)
        }
        forall(k: Nat) {
            if not k < n {
                coeff_zero_from_at(p.coeff, n, k) =
                    (not k < n implies coeff_zero_at(p.coeff, k))
                coeff_zero_at(p.coeff, k)
                coeff_zero_at(p.coeff, k) = (p.coeff(k) = R.0)
                p.coeff(k) = R.0
                semiring_hom_zero(f)
                polynomial_map_coeff_apply(f, p, k)
                polynomial_map(f, p).coeff(k) = S.0
            }
            coeff_zero_from_at(polynomial_map(f, p).coeff, n, k) =
                (not k < n implies coeff_zero_at(polynomial_map(f, p).coeff, k))
            coeff_zero_at(polynomial_map(f, p).coeff, k) =
                (polynomial_map(f, p).coeff(k) = S.0)
            coeff_zero_from_at(polynomial_map(f, p).coeff, n, k)
        }
        coeff_zero_from(polynomial_map(f, p).coeff, n) = forall(k: Nat) {
            coeff_zero_from_at(polynomial_map(f, p).coeff, n, k)
        }
        coeff_zero_from(polynomial_map(f, p).coeff, n)
        polynomial_support_bounded_by(polynomial_map(f, p), n) =
            coeff_zero_from(polynomial_map(f, p).coeff, n)
        polynomial_support_bounded_by(polynomial_map(f, p), n)
    }
}

/// Bounded polynomial evaluation commutes with a semiring homomorphism.
theorem polynomial_eval_bound_map[R: Semiring, S: Semiring](
    f: SemiringHom[R, S],
    p: Polynomial[R],
    x: R,
    n: Nat
) {
    polynomial_eval_bound(polynomial_map(f, p), f.hom(x), n) =
        f.hom(polynomial_eval_bound(p, x, n))
} by {
    polynomial_eval_bound(polynomial_map(f, p), f.hom(x), n) =
        coeff_eval(polynomial_map(f, p).coeff, f.hom(x), n)
    forall(k: Nat) {
        polynomial_map_coeff_apply(f, p, k)
        polynomial_map(f, p).coeff(k) = polynomial_map_coeff_fn(f, p.coeff, k)
    }
    function_extensionality(polynomial_map(f, p).coeff, polynomial_map_coeff_fn(f, p.coeff))
    coeff_eval_map(f, p.coeff, x, n)
    polynomial_eval_bound(p, x, n) = coeff_eval(p.coeff, x, n)
}

/// Support-independent polynomial evaluation commutes with a semiring homomorphism.
theorem polynomial_eval_map[R: Semiring, S: Semiring](
    f: SemiringHom[R, S],
    p: Polynomial[R],
    x: R
) {
    polynomial_eval(polynomial_map(f, p), f.hom(x)) = f.hom(polynomial_eval(p, x))
} by {
    polynomial_support_bound_exists(p)
    let n: Nat satisfy {
        polynomial_support_bounded_by(p, n)
    }
    polynomial_map_support_bounded_by(f, p, n)
    polynomial_eval_eq_eval_bound_of_support_bounded(p, x, n)
    polynomial_eval(p, x) = polynomial_eval_bound(p, x, n)
    polynomial_eval_eq_eval_bound_of_support_bounded(polynomial_map(f, p), f.hom(x), n)
    polynomial_eval(polynomial_map(f, p), f.hom(x)) =
        polynomial_eval_bound(polynomial_map(f, p), f.hom(x), n)
    polynomial_eval_bound_map(f, p, x, n)
    polynomial_eval_bound(polynomial_map(f, p), f.hom(x), n) =
        f.hom(polynomial_eval_bound(p, x, n))
    f.hom(polynomial_eval_bound(p, x, n)) = f.hom(polynomial_eval(p, x))
}

/// True if a point is a root of a polynomial.
define is_polynomial_root[R: Semiring](p: Polynomial[R], x: R) -> Bool {
    polynomial_eval(p, x) = R.0
}

/// True if a polynomial value vanishes after applying a semiring homomorphism.
define is_polynomial_root_mod_hom[R: Semiring, S: Semiring](
    f: SemiringHom[R, S],
    p: Polynomial[R],
    x: R
) -> Bool {
    f.hom(polynomial_eval(p, x)) = S.0
}

/// Vanishing modulo a semiring homomorphism is exactly vanishing of the
/// transported polynomial at the transported point.
theorem polynomial_root_mod_hom_iff_mapped_root[R: Semiring, S: Semiring](
    f: SemiringHom[R, S],
    p: Polynomial[R],
    x: R
) {
    is_polynomial_root_mod_hom(f, p, x) =
        is_polynomial_root(polynomial_map(f, p), f.hom(x))
} by {
    polynomial_eval_map(f, p, x)
    is_polynomial_root_mod_hom(f, p, x) =
        (f.hom(polynomial_eval(p, x)) = S.0)
    is_polynomial_root(polynomial_map(f, p), f.hom(x)) =
        (polynomial_eval(polynomial_map(f, p), f.hom(x)) = S.0)
}
