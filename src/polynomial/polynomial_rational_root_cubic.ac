/// The rational root theorem for cubics.
///
/// This file works in the interface universe of the polynomial module
/// (`from polynomial import ...`), because the coefficient-level rational
/// root theorem of `src/polynomial_rational_root_bridge.ac` is stated in
/// terms of the interface `coeff_eval`.
///
/// Contents: the cubic `a X^3 + b X^2 + c X + d` with integer coefficients
/// and its rationalised version, the rational root theorem for cubics (the
/// numerator divides the constant term, the denominator divides the leading
/// coefficient), and the concrete cubic `X^3 - 6 X^2 + 11 X - 6` with the
/// rational roots `1`, `2`, `3`, all dividing the constant term `6`.

from nat import Nat, lt_suc, lt_trans, lte_trans, add_suc_right, lt_or_lte,
    lt_and_lte, lt_not_symm, lt_not_ref, lte_imp_not_lt
from order import lt_imp_lte
from int import Int, sub_nat, add_sub_nat, sub_nat_zero_right, sub_nat_zero_left,
    sub_nat_add_left, sub_nat_add_right, neg_sub_nat, mul_from_nat, add_neg
from rat import Rat
from semiring import Semiring
from data.basic.functions import function_extensionality
from data.int.int_coprime import is_coprime, is_coprime_one_right
from data.rat.rat_int_hom import int_to_rat_hom, int_to_rat_hom_apply
from algebra.ring.ring import mul_neg_left, mul_neg_right
from algebra.add_group import inverse_inverse
from algebra.field.field import inverse_one
from polynomial import Polynomial, polynomial_constant, polynomial_monomial, polynomial_add_coeff,
    polynomial_monomial_coeff_of_ne, polynomial_monomial_coeff_self,
    polynomial_constant_coeff_zero, polynomial_constant_coeff_of_ne_zero,
    polynomial_constant_support_bounded_by_one, polynomial_add_support_bounded_by,
    polynomial_support_bounded_by, polynomial_support_bounded_by_monotone,
    polynomial_eval, polynomial_eval_bound, polynomial_eval_bound_eq_coeff_eval,
    polynomial_eval_eq_eval_bound_of_support_bounded, coeff_eval, coeff_tail,
    coeff_zero_at, coeff_zero_from, coeff_zero_from_at,
    polynomial_map, polynomial_map_coeff_apply, polynomial_map_support_bounded_by,
    polynomial_eval_map
from polynomial_rational_root_bridge import rat_coeffs, rat_coeffs_apply,
    rat_root_numerator_divides, rat_root_denominator_divides

numerals Nat
numerals Int
numerals Rat

/// One is at most four.
lemma nat_one_le_four {
    Nat.1 <= Nat.4
} by {
    lt_suc(Nat.1)
    Nat.1 < Nat.1.suc
    Nat.1.suc = Nat.2
    lt_imp_lte[Nat](Nat.1, Nat.2)
    Nat.1 <= Nat.2
    lt_suc(Nat.2)
    Nat.2 < Nat.2.suc
    Nat.2.suc = Nat.3
    lt_imp_lte[Nat](Nat.2, Nat.3)
    Nat.2 <= Nat.3
    lte_trans(Nat.1, Nat.2, Nat.3)
    Nat.1 <= Nat.3
    lt_suc(Nat.3)
    Nat.3 < Nat.3.suc
    Nat.3.suc = Nat.4
    lt_imp_lte[Nat](Nat.3, Nat.4)
    Nat.3 <= Nat.4
    lte_trans(Nat.1, Nat.3, Nat.4)
    Nat.1 <= Nat.4
}

/// Two is at most four.
lemma nat_two_le_four {
    Nat.2 <= Nat.4
} by {
    lt_suc(Nat.2)
    Nat.2 < Nat.2.suc
    Nat.2.suc = Nat.3
    lt_imp_lte[Nat](Nat.2, Nat.3)
    Nat.2 <= Nat.3
    lt_suc(Nat.3)
    Nat.3 < Nat.3.suc
    Nat.3.suc = Nat.4
    lt_imp_lte[Nat](Nat.3, Nat.4)
    Nat.3 <= Nat.4
    lte_trans(Nat.2, Nat.3, Nat.4)
    Nat.2 <= Nat.4
}

/// Three is at most four.
lemma nat_three_le_four {
    Nat.3 <= Nat.4
} by {
    lt_suc(Nat.3)
    Nat.3 < Nat.3.suc
    Nat.3.suc = Nat.4
    lt_imp_lte[Nat](Nat.3, Nat.4)
    Nat.3 <= Nat.4
}

// ---------------------------------------------------------------------------
// The cubic with integer coefficients, in the interface universe
// ---------------------------------------------------------------------------

/// The cubic polynomial `a * X^3 + b * X^2 + c * X + d`.
define int_cubic_polynomial(a: Int, b: Int, c: Int, d: Int) -> Polynomial[Int] {
    polynomial_constant(d) + polynomial_monomial(Nat.1, c) + polynomial_monomial(Nat.2, b) +
        polynomial_monomial(Nat.3, a)
}

/// The constant coefficient of a cubic is its constant term.
theorem cubic_coeff_zero(a: Int, b: Int, c: Int, d: Int) {
    int_cubic_polynomial(a, b, c, d).coeff(Nat.0) = d
} by {
    polynomial_add_coeff(polynomial_constant(d) + polynomial_monomial(Nat.1, c) +
        polynomial_monomial(Nat.2, b), polynomial_monomial(Nat.3, a), Nat.0)
    int_cubic_polynomial(a, b, c, d).coeff(Nat.0) =
        (polynomial_constant(d) + polynomial_monomial(Nat.1, c) +
            polynomial_monomial(Nat.2, b)).coeff(Nat.0) +
        polynomial_monomial(Nat.3, a).coeff(Nat.0)
    polynomial_add_coeff(polynomial_constant(d) + polynomial_monomial(Nat.1, c),
        polynomial_monomial(Nat.2, b), Nat.0)
    (polynomial_constant(d) + polynomial_monomial(Nat.1, c) +
        polynomial_monomial(Nat.2, b)).coeff(Nat.0) =
        (polynomial_constant(d) + polynomial_monomial(Nat.1, c)).coeff(Nat.0) +
        polynomial_monomial(Nat.2, b).coeff(Nat.0)
    polynomial_add_coeff(polynomial_constant(d), polynomial_monomial(Nat.1, c), Nat.0)
    (polynomial_constant(d) + polynomial_monomial(Nat.1, c)).coeff(Nat.0) =
        polynomial_constant(d).coeff(Nat.0) + polynomial_monomial(Nat.1, c).coeff(Nat.0)
    polynomial_constant_coeff_zero(d)
    polynomial_constant(d).coeff(Nat.0) = d
    Nat.0 != Nat.1
    polynomial_monomial_coeff_of_ne(Nat.1, c, Nat.0)
    polynomial_monomial(Nat.1, c).coeff(Nat.0) = Int.0
    Nat.0 != Nat.2
    polynomial_monomial_coeff_of_ne(Nat.2, b, Nat.0)
    polynomial_monomial(Nat.2, b).coeff(Nat.0) = Int.0
    Nat.0 != Nat.3
    polynomial_monomial_coeff_of_ne(Nat.3, a, Nat.0)
    polynomial_monomial(Nat.3, a).coeff(Nat.0) = Int.0
    int_cubic_polynomial(a, b, c, d).coeff(Nat.0) = d + Int.0 + Int.0 + Int.0
    d + Int.0 + Int.0 + Int.0 = d
    int_cubic_polynomial(a, b, c, d).coeff(Nat.0) = d
}

/// The linear coefficient of a cubic is its middle term.
theorem cubic_coeff_one(a: Int, b: Int, c: Int, d: Int) {
    int_cubic_polynomial(a, b, c, d).coeff(Nat.1) = c
} by {
    polynomial_add_coeff(polynomial_constant(d) + polynomial_monomial(Nat.1, c) +
        polynomial_monomial(Nat.2, b), polynomial_monomial(Nat.3, a), Nat.1)
    int_cubic_polynomial(a, b, c, d).coeff(Nat.1) =
        (polynomial_constant(d) + polynomial_monomial(Nat.1, c) +
            polynomial_monomial(Nat.2, b)).coeff(Nat.1) +
        polynomial_monomial(Nat.3, a).coeff(Nat.1)
    polynomial_add_coeff(polynomial_constant(d) + polynomial_monomial(Nat.1, c),
        polynomial_monomial(Nat.2, b), Nat.1)
    (polynomial_constant(d) + polynomial_monomial(Nat.1, c) +
        polynomial_monomial(Nat.2, b)).coeff(Nat.1) =
        (polynomial_constant(d) + polynomial_monomial(Nat.1, c)).coeff(Nat.1) +
        polynomial_monomial(Nat.2, b).coeff(Nat.1)
    polynomial_add_coeff(polynomial_constant(d), polynomial_monomial(Nat.1, c), Nat.1)
    (polynomial_constant(d) + polynomial_monomial(Nat.1, c)).coeff(Nat.1) =
        polynomial_constant(d).coeff(Nat.1) + polynomial_monomial(Nat.1, c).coeff(Nat.1)
    Nat.1 != Nat.0
    polynomial_constant_coeff_of_ne_zero(d, Nat.1)
    polynomial_constant(d).coeff(Nat.1) = Int.0
    polynomial_monomial_coeff_self(Nat.1, c)
    polynomial_monomial(Nat.1, c).coeff(Nat.1) = c
    Nat.1 != Nat.2
    polynomial_monomial_coeff_of_ne(Nat.2, b, Nat.1)
    polynomial_monomial(Nat.2, b).coeff(Nat.1) = Int.0
    Nat.1 != Nat.3
    polynomial_monomial_coeff_of_ne(Nat.3, a, Nat.1)
    polynomial_monomial(Nat.3, a).coeff(Nat.1) = Int.0
    int_cubic_polynomial(a, b, c, d).coeff(Nat.1) = Int.0 + c + Int.0 + Int.0
    Int.0 + c + Int.0 + Int.0 = c
    int_cubic_polynomial(a, b, c, d).coeff(Nat.1) = c
}

/// The quadratic coefficient of a cubic is its fourth term.
theorem cubic_coeff_two(a: Int, b: Int, c: Int, d: Int) {
    int_cubic_polynomial(a, b, c, d).coeff(Nat.2) = b
} by {
    polynomial_add_coeff(polynomial_constant(d) + polynomial_monomial(Nat.1, c) +
        polynomial_monomial(Nat.2, b), polynomial_monomial(Nat.3, a), Nat.2)
    int_cubic_polynomial(a, b, c, d).coeff(Nat.2) =
        (polynomial_constant(d) + polynomial_monomial(Nat.1, c) +
            polynomial_monomial(Nat.2, b)).coeff(Nat.2) +
        polynomial_monomial(Nat.3, a).coeff(Nat.2)
    polynomial_add_coeff(polynomial_constant(d) + polynomial_monomial(Nat.1, c),
        polynomial_monomial(Nat.2, b), Nat.2)
    (polynomial_constant(d) + polynomial_monomial(Nat.1, c) +
        polynomial_monomial(Nat.2, b)).coeff(Nat.2) =
        (polynomial_constant(d) + polynomial_monomial(Nat.1, c)).coeff(Nat.2) +
        polynomial_monomial(Nat.2, b).coeff(Nat.2)
    polynomial_add_coeff(polynomial_constant(d), polynomial_monomial(Nat.1, c), Nat.2)
    (polynomial_constant(d) + polynomial_monomial(Nat.1, c)).coeff(Nat.2) =
        polynomial_constant(d).coeff(Nat.2) + polynomial_monomial(Nat.1, c).coeff(Nat.2)
    Nat.2 != Nat.0
    polynomial_constant_coeff_of_ne_zero(d, Nat.2)
    polynomial_constant(d).coeff(Nat.2) = Int.0
    Nat.2 != Nat.1
    polynomial_monomial_coeff_of_ne(Nat.1, c, Nat.2)
    polynomial_monomial(Nat.1, c).coeff(Nat.2) = Int.0
    polynomial_monomial_coeff_self(Nat.2, b)
    polynomial_monomial(Nat.2, b).coeff(Nat.2) = b
    Nat.2 != Nat.3
    polynomial_monomial_coeff_of_ne(Nat.3, a, Nat.2)
    polynomial_monomial(Nat.3, a).coeff(Nat.2) = Int.0
    int_cubic_polynomial(a, b, c, d).coeff(Nat.2) = Int.0 + Int.0 + b + Int.0
    Int.0 + Int.0 + b + Int.0 = b
    int_cubic_polynomial(a, b, c, d).coeff(Nat.2) = b
}

/// The cubic coefficient of a cubic is its leading term.
theorem cubic_coeff_three(a: Int, b: Int, c: Int, d: Int) {
    int_cubic_polynomial(a, b, c, d).coeff(Nat.3) = a
} by {
    polynomial_add_coeff(polynomial_constant(d) + polynomial_monomial(Nat.1, c) +
        polynomial_monomial(Nat.2, b), polynomial_monomial(Nat.3, a), Nat.3)
    int_cubic_polynomial(a, b, c, d).coeff(Nat.3) =
        (polynomial_constant(d) + polynomial_monomial(Nat.1, c) +
            polynomial_monomial(Nat.2, b)).coeff(Nat.3) +
        polynomial_monomial(Nat.3, a).coeff(Nat.3)
    polynomial_add_coeff(polynomial_constant(d) + polynomial_monomial(Nat.1, c),
        polynomial_monomial(Nat.2, b), Nat.3)
    (polynomial_constant(d) + polynomial_monomial(Nat.1, c) +
        polynomial_monomial(Nat.2, b)).coeff(Nat.3) =
        (polynomial_constant(d) + polynomial_monomial(Nat.1, c)).coeff(Nat.3) +
        polynomial_monomial(Nat.2, b).coeff(Nat.3)
    polynomial_add_coeff(polynomial_constant(d), polynomial_monomial(Nat.1, c), Nat.3)
    (polynomial_constant(d) + polynomial_monomial(Nat.1, c)).coeff(Nat.3) =
        polynomial_constant(d).coeff(Nat.3) + polynomial_monomial(Nat.1, c).coeff(Nat.3)
    Nat.3 != Nat.0
    polynomial_constant_coeff_of_ne_zero(d, Nat.3)
    polynomial_constant(d).coeff(Nat.3) = Int.0
    Nat.3 != Nat.1
    polynomial_monomial_coeff_of_ne(Nat.1, c, Nat.3)
    polynomial_monomial(Nat.1, c).coeff(Nat.3) = Int.0
    Nat.3 != Nat.2
    polynomial_monomial_coeff_of_ne(Nat.2, b, Nat.3)
    polynomial_monomial(Nat.2, b).coeff(Nat.3) = Int.0
    polynomial_monomial_coeff_self(Nat.3, a)
    polynomial_monomial(Nat.3, a).coeff(Nat.3) = a
    int_cubic_polynomial(a, b, c, d).coeff(Nat.3) = Int.0 + Int.0 + Int.0 + a
    Int.0 + Int.0 + Int.0 + a = a
    int_cubic_polynomial(a, b, c, d).coeff(Nat.3) = a
}

/// A monomial is support-bounded by one more than its exponent.
theorem monomial_support_bounded_by_suc(n: Nat, r: Int) {
    polynomial_support_bounded_by(Polynomial[Int].monomial(n, r), n.suc)
} by {
    forall(k: Nat) {
        if k < n.suc {
            coeff_zero_from_at(Polynomial[Int].monomial(n, r).coeff, n.suc, k) =
                (not k < n.suc implies coeff_zero_at(Polynomial[Int].monomial(n, r).coeff, k))
            coeff_zero_from_at(Polynomial[Int].monomial(n, r).coeff, n.suc, k)
        } else {
            not k < n.suc
            lt_or_lte(k, n.suc)
            n.suc <= k
            if k = n {
                lt_suc(n)
                n < n.suc
                lt_and_lte(n, n.suc, k)
                n < k
                lt_not_symm(n, k)
                not k < n
                lt_and_lte(k, n, k)
                k < k
                lt_not_symm(k, k)
                not k < k
                false
            }
            k != n
            polynomial_monomial_coeff_of_ne(n, r, k)
            Polynomial[Int].monomial(n, r).coeff(k) = Int.0
            coeff_zero_at(Polynomial[Int].monomial(n, r).coeff, k)
            coeff_zero_from_at(Polynomial[Int].monomial(n, r).coeff, n.suc, k) =
                (not k < n.suc implies coeff_zero_at(Polynomial[Int].monomial(n, r).coeff, k))
            coeff_zero_from_at(Polynomial[Int].monomial(n, r).coeff, n.suc, k)
        }
    }
    coeff_zero_from(Polynomial[Int].monomial(n, r).coeff, n.suc) = forall(k: Nat) {
        coeff_zero_from_at(Polynomial[Int].monomial(n, r).coeff, n.suc, k)
    }
    coeff_zero_from(Polynomial[Int].monomial(n, r).coeff, n.suc)
    polynomial_support_bounded_by(Polynomial[Int].monomial(n, r), n.suc) =
        coeff_zero_from(Polynomial[Int].monomial(n, r).coeff, n.suc)
    polynomial_support_bounded_by(Polynomial[Int].monomial(n, r), n.suc)
}

/// A cubic polynomial is supported below four coefficient slots.
theorem cubic_support_bounded_by_four(a: Int, b: Int, c: Int, d: Int) {
    polynomial_support_bounded_by(int_cubic_polynomial(a, b, c, d), Nat.4)
} by {
    polynomial_constant_support_bounded_by_one(d)
    polynomial_support_bounded_by(polynomial_constant(d), Nat.1)
    nat_one_le_four
    polynomial_support_bounded_by_monotone(polynomial_constant(d), Nat.1, Nat.4)
    polynomial_support_bounded_by(polynomial_constant(d), Nat.4)
    monomial_support_bounded_by_suc(Nat.1, c)
    polynomial_support_bounded_by(polynomial_monomial(Nat.1, c), Nat.2)
    nat_two_le_four
    polynomial_support_bounded_by_monotone(polynomial_monomial(Nat.1, c), Nat.2, Nat.4)
    polynomial_support_bounded_by(polynomial_monomial(Nat.1, c), Nat.4)
    polynomial_add_support_bounded_by(polynomial_constant(d), polynomial_monomial(Nat.1, c), Nat.4)
    polynomial_support_bounded_by(polynomial_constant(d) + polynomial_monomial(Nat.1, c), Nat.4)
    monomial_support_bounded_by_suc(Nat.2, b)
    polynomial_support_bounded_by(polynomial_monomial(Nat.2, b), Nat.3)
    nat_three_le_four
    polynomial_support_bounded_by_monotone(polynomial_monomial(Nat.2, b), Nat.3, Nat.4)
    polynomial_support_bounded_by(polynomial_monomial(Nat.2, b), Nat.4)
    polynomial_add_support_bounded_by(polynomial_constant(d) + polynomial_monomial(Nat.1, c),
        polynomial_monomial(Nat.2, b), Nat.4)
    polynomial_support_bounded_by(
        (polynomial_constant(d) + polynomial_monomial(Nat.1, c)) + polynomial_monomial(Nat.2, b),
        Nat.4)
    monomial_support_bounded_by_suc(Nat.3, a)
    polynomial_support_bounded_by(polynomial_monomial(Nat.3, a), Nat.4)
    polynomial_add_support_bounded_by(
        (polynomial_constant(d) + polynomial_monomial(Nat.1, c)) + polynomial_monomial(Nat.2, b),
        polynomial_monomial(Nat.3, a), Nat.4)
    polynomial_support_bounded_by(int_cubic_polynomial(a, b, c, d), Nat.4)
}

// ---------------------------------------------------------------------------
// The rational root theorem for cubics
// ---------------------------------------------------------------------------

/// The rationalised cubic `a X^3 + b X^2 + c X + d`: the integer cubic with
/// every coefficient embedded into the rationals.
define rat_cubic_polynomial(a: Int, b: Int, c: Int, d: Int) -> Polynomial[Rat] {
    polynomial_map(int_to_rat_hom, int_cubic_polynomial(a, b, c, d))
}

/// A rationalised coefficient is the embedding of the integer one.
theorem rat_cubic_coeff_apply(a: Int, b: Int, c: Int, d: Int, i: Nat) {
    rat_cubic_polynomial(a, b, c, d).coeff(i) = Rat.from_int(int_cubic_polynomial(a, b, c, d).coeff(i))
} by {
    polynomial_map_coeff_apply(int_to_rat_hom, int_cubic_polynomial(a, b, c, d), i)
    polynomial_map(int_to_rat_hom, int_cubic_polynomial(a, b, c, d)).coeff(i) =
        int_to_rat_hom.hom(int_cubic_polynomial(a, b, c, d).coeff(i))
    int_to_rat_hom_apply(int_cubic_polynomial(a, b, c, d).coeff(i))
    int_to_rat_hom.hom(int_cubic_polynomial(a, b, c, d).coeff(i)) =
        Rat.from_int(int_cubic_polynomial(a, b, c, d).coeff(i))
    rat_cubic_polynomial(a, b, c, d).coeff(i) =
        Rat.from_int(int_cubic_polynomial(a, b, c, d).coeff(i))
}

/// The transport lemma for the rational root theorem for cubics: a rational
/// root of the rationalised cubic `a X^3 + b X^2 + c X + d` gives the
/// coefficient-level root condition
/// `coeff_eval(rat_coeffs(...), p / q, 4) = 0` at bound four.
///
/// The proof transports the root condition from global polynomial evaluation
/// to bounded coefficient evaluation and identifies the rationalised
/// coefficients with the rational embedding of the integer ones.
theorem cubic_rat_root_coeff_eval_zero(a: Int, b: Int, c: Int, d: Int, p: Int, q: Int) {
    q != Int.0 and is_coprime(p, q) and
    polynomial_eval(rat_cubic_polynomial(a, b, c, d),
        Rat.from_int(p) / Rat.from_int(q)) = Rat.0
    implies coeff_eval(rat_coeffs(int_cubic_polynomial(a, b, c, d).coeff),
        Rat.from_int(p) / Rat.from_int(q), Nat.3.suc) = Rat.0
} by {
    if q != Int.0 and is_coprime(p, q) and
        polynomial_eval(rat_cubic_polynomial(a, b, c, d),
            Rat.from_int(p) / Rat.from_int(q)) = Rat.0 {
        cubic_support_bounded_by_four(a, b, c, d)
        polynomial_support_bounded_by(int_cubic_polynomial(a, b, c, d), Nat.3.suc)
        polynomial_map_support_bounded_by(int_to_rat_hom, int_cubic_polynomial(a, b, c, d), Nat.3.suc)
        polynomial_support_bounded_by(polynomial_map(int_to_rat_hom, int_cubic_polynomial(a, b, c, d)), Nat.3.suc)
        polynomial_support_bounded_by(rat_cubic_polynomial(a, b, c, d), Nat.3.suc)
        polynomial_eval_eq_eval_bound_of_support_bounded(rat_cubic_polynomial(a, b, c, d),
            Rat.from_int(p) / Rat.from_int(q), Nat.3.suc)
        polynomial_eval(rat_cubic_polynomial(a, b, c, d),
            Rat.from_int(p) / Rat.from_int(q)) =
            polynomial_eval_bound(rat_cubic_polynomial(a, b, c, d),
                Rat.from_int(p) / Rat.from_int(q), Nat.3.suc)
        polynomial_eval_bound(rat_cubic_polynomial(a, b, c, d),
            Rat.from_int(p) / Rat.from_int(q), Nat.3.suc) = Rat.0
        polynomial_eval_bound_eq_coeff_eval(rat_cubic_polynomial(a, b, c, d),
            Rat.from_int(p) / Rat.from_int(q), Nat.3.suc)
        polynomial_eval_bound(rat_cubic_polynomial(a, b, c, d),
            Rat.from_int(p) / Rat.from_int(q), Nat.3.suc) =
            coeff_eval(rat_cubic_polynomial(a, b, c, d).coeff,
                Rat.from_int(p) / Rat.from_int(q), Nat.3.suc)
        coeff_eval(rat_cubic_polynomial(a, b, c, d).coeff,
            Rat.from_int(p) / Rat.from_int(q), Nat.3.suc) = Rat.0
        forall(i: Nat) {
            rat_cubic_coeff_apply(a, b, c, d, i)
            rat_cubic_polynomial(a, b, c, d).coeff(i) =
                Rat.from_int(int_cubic_polynomial(a, b, c, d).coeff(i))
            rat_coeffs_apply(int_cubic_polynomial(a, b, c, d).coeff, i)
            rat_coeffs(int_cubic_polynomial(a, b, c, d).coeff)(i) =
                Rat.from_int(int_cubic_polynomial(a, b, c, d).coeff(i))
            rat_cubic_polynomial(a, b, c, d).coeff(i) =
                rat_coeffs(int_cubic_polynomial(a, b, c, d).coeff)(i)
        }
        function_extensionality[Nat, Rat](rat_cubic_polynomial(a, b, c, d).coeff,
            rat_coeffs(int_cubic_polynomial(a, b, c, d).coeff))
        rat_cubic_polynomial(a, b, c, d).coeff =
            rat_coeffs(int_cubic_polynomial(a, b, c, d).coeff)
        coeff_eval(rat_cubic_polynomial(a, b, c, d).coeff,
            Rat.from_int(p) / Rat.from_int(q), Nat.3.suc) =
            coeff_eval(rat_coeffs(int_cubic_polynomial(a, b, c, d).coeff),
                Rat.from_int(p) / Rat.from_int(q), Nat.3.suc)
        coeff_eval(rat_coeffs(int_cubic_polynomial(a, b, c, d).coeff),
            Rat.from_int(p) / Rat.from_int(q), Nat.3.suc) = Rat.0
    }
}

/// The coefficient-level rational root theorem for the denominator: the
/// denominator of a rational root in lowest terms divides the leading
/// coefficient.
theorem cubic_rat_root_denominator_divide(c: Nat -> Int, a: Int, b: Int, n: Nat) {
    b != Int.0 and is_coprime(a, b) and
        coeff_eval(rat_coeffs(c), Rat.from_int(a) / Rat.from_int(b), n.suc) = Rat.0
    implies b.divides(c(n))
} by {
    if b != Int.0 and is_coprime(a, b) and
        coeff_eval(rat_coeffs(c), Rat.from_int(a) / Rat.from_int(b), n.suc) = Rat.0 {
        rat_root_denominator_divides(c, a, b, n)
        b.divides(c(n))
    }
}

/// The coefficient-level rational root theorem for the numerator: the
/// numerator of a rational root in lowest terms divides the constant
/// coefficient.
theorem cubic_rat_root_numerator_divide(c: Nat -> Int, a: Int, b: Int, n: Nat) {
    b != Int.0 and is_coprime(a, b) and
        coeff_eval(rat_coeffs(c), Rat.from_int(a) / Rat.from_int(b), n.suc) = Rat.0
    implies a.divides(c(Nat.0))
} by {
    if b != Int.0 and is_coprime(a, b) and
        coeff_eval(rat_coeffs(c), Rat.from_int(a) / Rat.from_int(b), n.suc) = Rat.0 {
        rat_root_numerator_divides(c, a, b, n)
        a.divides(c(Nat.0))
    }
}


/// The rational root theorem for cubics, first half: the numerator of a
/// rational root of `a X^3 + b X^2 + c X + d` in lowest terms divides the
/// constant term `d`.
theorem cubic_rational_root_numerator_divides(a: Int, b: Int, c: Int, d: Int, p: Int, q: Int) {
    q != Int.0 and is_coprime(p, q) and
    polynomial_eval(rat_cubic_polynomial(a, b, c, d),
        Rat.from_int(p) / Rat.from_int(q)) = Rat.0
    implies p.divides(d)
} by {
    if q != Int.0 and is_coprime(p, q) and
        polynomial_eval(rat_cubic_polynomial(a, b, c, d),
            Rat.from_int(p) / Rat.from_int(q)) = Rat.0 {
        cubic_rat_root_coeff_eval_zero(a, b, c, d, p, q)
        coeff_eval(rat_coeffs(int_cubic_polynomial(a, b, c, d).coeff),
            Rat.from_int(p) / Rat.from_int(q), Nat.3.suc) = Rat.0
        cubic_coeff_zero(a, b, c, d)
        int_cubic_polynomial(a, b, c, d).coeff(Nat.0) = d
        cubic_rat_root_numerator_divide(int_cubic_polynomial(a, b, c, d).coeff, p, q, Nat.3)
        p.divides(int_cubic_polynomial(a, b, c, d).coeff(Nat.0))
        p.divides(d)
    }
}

/// The rational root theorem for cubics, second half: the denominator of a
/// rational root of `a X^3 + b X^2 + c X + d` in lowest terms divides the
/// leading coefficient `a`.
theorem cubic_rational_root_denominator_divides(a: Int, b: Int, c: Int, d: Int, p: Int, q: Int) {
    q != Int.0 and is_coprime(p, q) and
    polynomial_eval(rat_cubic_polynomial(a, b, c, d),
        Rat.from_int(p) / Rat.from_int(q)) = Rat.0
    implies q.divides(a)
} by {
    if q != Int.0 and is_coprime(p, q) and
        polynomial_eval(rat_cubic_polynomial(a, b, c, d),
            Rat.from_int(p) / Rat.from_int(q)) = Rat.0 {
        cubic_rat_root_coeff_eval_zero(a, b, c, d, p, q)
        coeff_eval(rat_coeffs(int_cubic_polynomial(a, b, c, d).coeff),
            Rat.from_int(p) / Rat.from_int(q), Nat.3.suc) = Rat.0
        cubic_coeff_three(a, b, c, d)
        int_cubic_polynomial(a, b, c, d).coeff(Nat.3) = a
        cubic_rat_root_denominator_divide(int_cubic_polynomial(a, b, c, d).coeff, p, q, Nat.3)
        q.divides(int_cubic_polynomial(a, b, c, d).coeff(Nat.3))
        q.divides(a)
    }
}

// ---------------------------------------------------------------------------
// Concrete integer arithmetic for the cubic `X^3 - 6 X^2 + 11 X - 6`
// ---------------------------------------------------------------------------

/// Bounded evaluation at bound four, unfolded to the Horner form.
lemma coeff_eval_four(c: Nat -> Int, x: Int) {
    coeff_eval(c, x, Nat.4) = c(Nat.0) + x * (c(Nat.1) + x * (c(Nat.2) + x * (c(Nat.3) + x * Int.0)))
} by {
    coeff_eval(c, x, Nat.4) = c(Nat.0) + x * coeff_eval(coeff_tail(c), x, Nat.3)
    coeff_eval(coeff_tail(c), x, Nat.3) = c(Nat.1) + x * coeff_eval(coeff_tail(coeff_tail(c)), x, Nat.2)
    coeff_tail(coeff_tail(c))(Nat.0) = c(Nat.2)
    coeff_eval(coeff_tail(coeff_tail(c)), x, Nat.2) =
        c(Nat.2) + x * coeff_eval(coeff_tail(coeff_tail(coeff_tail(c))), x, Nat.1)
    coeff_tail(coeff_tail(coeff_tail(c)))(Nat.0) = coeff_tail(coeff_tail(c))(Nat.0.suc)
    coeff_tail(coeff_tail(c))(Nat.0.suc) = coeff_tail(c)(Nat.0.suc.suc)
    coeff_tail(c)(Nat.0.suc.suc) = c(Nat.0.suc.suc.suc)
    coeff_tail(c)(Nat.0.suc.suc) = c(Nat.3)
    coeff_tail(coeff_tail(coeff_tail(c)))(Nat.0) = c(Nat.3)
    coeff_eval(coeff_tail(coeff_tail(coeff_tail(c))), x, Nat.1) =
        c(Nat.3) + x * coeff_eval(coeff_tail(coeff_tail(coeff_tail(coeff_tail(c)))), x, Nat.0)
    coeff_eval(coeff_tail(coeff_tail(coeff_tail(coeff_tail(c)))), x, Nat.0) = Int.0
    coeff_eval(c, x, Nat.4) = c(Nat.0) + x * (c(Nat.1) + x * (c(Nat.2) + x * (c(Nat.3) + x * Int.0)))
}

/// `-from_nat(m) + from_nat(n) = sub_nat(n, m)`.
theorem int_neg_from_nat_add(n: Nat, m: Nat) {
    -Int.from_nat(m) + Int.from_nat(n) = sub_nat(n, m)
} by {
    sub_nat_zero_left(m)
    -Int.from_nat(m) = sub_nat(Nat.0, m)
    sub_nat_zero_right(n)
    Int.from_nat(n) = sub_nat(n, Nat.0)
    add_sub_nat(Nat.0, m, n, Nat.0)
    sub_nat(Nat.0, m) + sub_nat(n, Nat.0) = sub_nat(Nat.0 + n, m + Nat.0)
    sub_nat(Nat.0 + n, m + Nat.0) = sub_nat(n, m)
    -Int.from_nat(m) + Int.from_nat(n) = sub_nat(n, m)
}

/// `-from_nat(m) + from_nat(n) = from_nat(k)` when `m + k = n`.
theorem int_neg_from_nat_add_cancel(m: Nat, n: Nat, k: Nat) {
    m + k = n implies -Int.from_nat(m) + Int.from_nat(n) = Int.from_nat(k)
} by {
    if m + k = n {
        int_neg_from_nat_add(n, m)
        -Int.from_nat(m) + Int.from_nat(n) = sub_nat(n, m)
        sub_nat_add_left(k, m)
        sub_nat(m + k, m) = Int.from_nat(k)
        m + k = n
        sub_nat(n, m) = Int.from_nat(k)
        -Int.from_nat(m) + Int.from_nat(n) = Int.from_nat(k)
    }
}

/// `-from_nat(m) + from_nat(n) = -from_nat(k)` when `n + k = m`.
theorem int_neg_from_nat_add_neg(m: Nat, n: Nat, k: Nat) {
    n + k = m implies -Int.from_nat(m) + Int.from_nat(n) = -Int.from_nat(k)
} by {
    if n + k = m {
        int_neg_from_nat_add(n, m)
        -Int.from_nat(m) + Int.from_nat(n) = sub_nat(n, m)
        neg_sub_nat(n, m)
        sub_nat(n, m) = -(sub_nat(m, n))
        sub_nat_add_left(k, n)
        sub_nat(n + k, n) = Int.from_nat(k)
        n + k = m
        sub_nat(m, n) = Int.from_nat(k)
        -(sub_nat(m, n)) = -Int.from_nat(k)
        sub_nat(n, m) = -Int.from_nat(k)
        -Int.from_nat(m) + Int.from_nat(n) = -Int.from_nat(k)
    }
}

/// `-6 + 2 = -4`.
theorem int_neg_six_add_two {
    -Int.6 + Int.2 = -Int.4
} by {
    int_neg_from_nat_add_neg(Nat.6, Nat.2, Nat.4)
    Nat.2 + Nat.4 = Nat.6
    -Int.from_nat(Nat.6) + Int.from_nat(Nat.2) = -Int.from_nat(Nat.4)
    Int.6 = Int.from_nat(Nat.6)
    Int.2 = Int.from_nat(Nat.2)
    Int.4 = Int.from_nat(Nat.4)
    -Int.6 + Int.2 = -Int.4
}

/// `11 + -8 = 3`.
theorem int_eleven_add_neg_eight {
    Int.11 + -Int.8 = Int.3
} by {
    int_neg_from_nat_add_cancel(Nat.8, Nat.11, Nat.3)
    Nat.3 = Nat.2.suc
    Nat.8 + Nat.3 = Nat.8 + Nat.2.suc
    add_suc_right(Nat.8, Nat.2)
    Nat.8 + Nat.2.suc = (Nat.8 + Nat.2).suc
    Nat.8 + Nat.2 = Nat.10
    (Nat.8 + Nat.2).suc = Nat.10.suc
    Nat.10.suc = Nat.11
    Nat.8 + Nat.3 = Nat.11
    -Int.from_nat(Nat.8) + Int.from_nat(Nat.11) = Int.from_nat(Nat.3)
    Int.11 = Int.from_nat(Nat.11)
    Int.8 = Int.from_nat(Nat.8)
    Int.3 = Int.from_nat(Nat.3)
    Int.11 + -Int.8 = Int.3
}

/// `2 * -4 = -8`.
theorem int_two_mul_neg_four {
    Int.2 * -Int.4 = -Int.8
} by {
    mul_neg_right(Int.2, Int.4)
    Int.2 * -Int.4 = -(Int.2 * Int.4)
    mul_from_nat(Nat.2, Nat.4)
    Int.2 * Int.4 = Int.from_nat(Nat.2 * Nat.4)
    Nat.2 * Nat.4 = Nat.8
    Int.from_nat(Nat.2 * Nat.4) = Int.from_nat(Nat.8)
    Int.from_nat(Nat.8) = Int.8
    Int.2 * Int.4 = Int.8
    -(Int.2 * Int.4) = -Int.8
    Int.2 * -Int.4 = -Int.8
}

/// `2 * 3 = 6`.
theorem int_two_mul_three {
    Int.2 * Int.3 = Int.6
} by {
    mul_from_nat(Nat.2, Nat.3)
    Int.2 * Int.3 = Int.from_nat(Nat.2 * Nat.3)
    Nat.2 * Nat.3 = Nat.6
    Int.from_nat(Nat.2 * Nat.3) = Int.from_nat(Nat.6)
    Int.from_nat(Nat.6) = Int.6
    Int.2 * Int.3 = Int.6
}

/// `-6 + 3 = -3`.
theorem int_neg_six_add_three {
    -Int.6 + Int.3 = -Int.3
} by {
    int_neg_from_nat_add_neg(Nat.6, Nat.3, Nat.3)
    Nat.3 + Nat.3 = Nat.6
    -Int.from_nat(Nat.6) + Int.from_nat(Nat.3) = -Int.from_nat(Nat.3)
    Int.6 = Int.from_nat(Nat.6)
    Int.3 = Int.from_nat(Nat.3)
    -Int.6 + Int.3 = -Int.3
}

/// `11 + -9 = 2`.
theorem int_eleven_add_neg_nine {
    Int.11 + -Int.9 = Int.2
} by {
    int_neg_from_nat_add_cancel(Nat.9, Nat.11, Nat.2)
    Nat.2 = Nat.1.suc
    Nat.9 + Nat.2 = Nat.9 + Nat.1.suc
    add_suc_right(Nat.9, Nat.1)
    Nat.9 + Nat.1.suc = (Nat.9 + Nat.1).suc
    Nat.9 + Nat.1 = Nat.10
    (Nat.9 + Nat.1).suc = Nat.10.suc
    Nat.10.suc = Nat.11
    Nat.9 + Nat.2 = Nat.11
    -Int.from_nat(Nat.9) + Int.from_nat(Nat.11) = Int.from_nat(Nat.2)
    Int.11 = Int.from_nat(Nat.11)
    Int.9 = Int.from_nat(Nat.9)
    Int.2 = Int.from_nat(Nat.2)
    Int.11 + -Int.9 = Int.2
}

/// `3 * -3 = -9`.
theorem int_three_mul_neg_three {
    Int.3 * -Int.3 = -Int.9
} by {
    mul_neg_right(Int.3, Int.3)
    Int.3 * -Int.3 = -(Int.3 * Int.3)
    mul_from_nat(Nat.3, Nat.3)
    Int.3 * Int.3 = Int.from_nat(Nat.3 * Nat.3)
    Nat.3 * Nat.3 = Nat.9
    Int.from_nat(Nat.3 * Nat.3) = Int.from_nat(Nat.9)
    Int.from_nat(Nat.9) = Int.9
    Int.3 * Int.3 = Int.9
    -(Int.3 * Int.3) = -Int.9
    Int.3 * -Int.3 = -Int.9
}

/// `3 * 2 = 6`.
theorem int_three_mul_two {
    Int.3 * Int.2 = Int.6
} by {
    mul_from_nat(Nat.3, Nat.2)
    Int.3 * Int.2 = Int.from_nat(Nat.3 * Nat.2)
    Nat.3 * Nat.2 = Nat.6
    Int.from_nat(Nat.3 * Nat.2) = Int.from_nat(Nat.6)
    Int.from_nat(Nat.6) = Int.6
    Int.3 * Int.2 = Int.6
}

/// The cubic `X^3 - 6 X^2 + 11 X - 6` with integer coefficients.
let int_cubic_x3_minus_6x2_plus_11x_minus_6: Polynomial[Int] =
    int_cubic_polynomial(Int.1, -Int.6, Int.11, -Int.6)

/// The integer cubic `X^3 - 6 X^2 + 11 X - 6` has the integer root `2`.
///
/// The value is computed in the Horner form at bound four.
theorem int_cubic_root_two {
    polynomial_eval(int_cubic_x3_minus_6x2_plus_11x_minus_6, Int.2) = Int.0
} by {
    cubic_support_bounded_by_four(Int.1, -Int.6, Int.11, -Int.6)
    polynomial_support_bounded_by(int_cubic_x3_minus_6x2_plus_11x_minus_6, Nat.4)
    polynomial_eval_eq_eval_bound_of_support_bounded(int_cubic_x3_minus_6x2_plus_11x_minus_6, Int.2, Nat.4)
    polynomial_eval(int_cubic_x3_minus_6x2_plus_11x_minus_6, Int.2) =
        polynomial_eval_bound(int_cubic_x3_minus_6x2_plus_11x_minus_6, Int.2, Nat.4)
    polynomial_eval_bound_eq_coeff_eval(int_cubic_x3_minus_6x2_plus_11x_minus_6, Int.2, Nat.4)
    polynomial_eval_bound(int_cubic_x3_minus_6x2_plus_11x_minus_6, Int.2, Nat.4) =
        coeff_eval(int_cubic_x3_minus_6x2_plus_11x_minus_6.coeff, Int.2, Nat.4)
    coeff_eval_four(int_cubic_x3_minus_6x2_plus_11x_minus_6.coeff, Int.2)
    coeff_eval(int_cubic_x3_minus_6x2_plus_11x_minus_6.coeff, Int.2, Nat.4) =
        int_cubic_x3_minus_6x2_plus_11x_minus_6.coeff(Nat.0) +
        Int.2 * (int_cubic_x3_minus_6x2_plus_11x_minus_6.coeff(Nat.1) +
            Int.2 * (int_cubic_x3_minus_6x2_plus_11x_minus_6.coeff(Nat.2) +
                Int.2 * (int_cubic_x3_minus_6x2_plus_11x_minus_6.coeff(Nat.3) + Int.2 * Int.0)))
    cubic_coeff_zero(Int.1, -Int.6, Int.11, -Int.6)
    int_cubic_x3_minus_6x2_plus_11x_minus_6.coeff(Nat.0) = -Int.6
    cubic_coeff_one(Int.1, -Int.6, Int.11, -Int.6)
    int_cubic_x3_minus_6x2_plus_11x_minus_6.coeff(Nat.1) = Int.11
    cubic_coeff_two(Int.1, -Int.6, Int.11, -Int.6)
    int_cubic_x3_minus_6x2_plus_11x_minus_6.coeff(Nat.2) = -Int.6
    cubic_coeff_three(Int.1, -Int.6, Int.11, -Int.6)
    int_cubic_x3_minus_6x2_plus_11x_minus_6.coeff(Nat.3) = Int.1
    coeff_eval(int_cubic_x3_minus_6x2_plus_11x_minus_6.coeff, Int.2, Nat.4) =
        -Int.6 + Int.2 * (Int.11 + Int.2 * (-Int.6 + Int.2 * (Int.1 + Int.2 * Int.0)))
    -Int.6 + Int.2 * (Int.11 + Int.2 * (-Int.6 + Int.2 * (Int.1 + Int.2 * Int.0))) =
        -Int.6 + Int.2 * (Int.11 + Int.2 * (-Int.6 + Int.2 * Int.1))
    int_neg_six_add_two
    -Int.6 + Int.2 = -Int.4
    -Int.6 + Int.2 * (Int.11 + Int.2 * (-Int.6 + Int.2 * Int.1)) =
        -Int.6 + Int.2 * (Int.11 + Int.2 * -Int.4)
    int_two_mul_neg_four
    Int.2 * -Int.4 = -Int.8
    -Int.6 + Int.2 * (Int.11 + Int.2 * -Int.4) = -Int.6 + Int.2 * (Int.11 + -Int.8)
    int_eleven_add_neg_eight
    Int.11 + -Int.8 = Int.3
    -Int.6 + Int.2 * (Int.11 + -Int.8) = -Int.6 + Int.2 * Int.3
    int_two_mul_three
    Int.2 * Int.3 = Int.6
    -Int.6 + Int.2 * Int.3 = -Int.6 + Int.6
    add_neg(Int.6)
    Int.6 + -Int.6 = Int.0
    -Int.6 + Int.6 = Int.0
    -Int.6 + Int.2 * Int.3 = Int.0
    coeff_eval(int_cubic_x3_minus_6x2_plus_11x_minus_6.coeff, Int.2, Nat.4) = Int.0
    polynomial_eval(int_cubic_x3_minus_6x2_plus_11x_minus_6, Int.2) = Int.0
}

/// The integer cubic `X^3 - 6 X^2 + 11 X - 6` has the integer root `3`.
///
/// The value is computed in the Horner form at bound four.
theorem int_cubic_root_three {
    polynomial_eval(int_cubic_x3_minus_6x2_plus_11x_minus_6, Int.3) = Int.0
} by {
    cubic_support_bounded_by_four(Int.1, -Int.6, Int.11, -Int.6)
    polynomial_support_bounded_by(int_cubic_x3_minus_6x2_plus_11x_minus_6, Nat.4)
    polynomial_eval_eq_eval_bound_of_support_bounded(int_cubic_x3_minus_6x2_plus_11x_minus_6, Int.3, Nat.4)
    polynomial_eval(int_cubic_x3_minus_6x2_plus_11x_minus_6, Int.3) =
        polynomial_eval_bound(int_cubic_x3_minus_6x2_plus_11x_minus_6, Int.3, Nat.4)
    polynomial_eval_bound_eq_coeff_eval(int_cubic_x3_minus_6x2_plus_11x_minus_6, Int.3, Nat.4)
    polynomial_eval_bound(int_cubic_x3_minus_6x2_plus_11x_minus_6, Int.3, Nat.4) =
        coeff_eval(int_cubic_x3_minus_6x2_plus_11x_minus_6.coeff, Int.3, Nat.4)
    coeff_eval_four(int_cubic_x3_minus_6x2_plus_11x_minus_6.coeff, Int.3)
    coeff_eval(int_cubic_x3_minus_6x2_plus_11x_minus_6.coeff, Int.3, Nat.4) =
        int_cubic_x3_minus_6x2_plus_11x_minus_6.coeff(Nat.0) +
        Int.3 * (int_cubic_x3_minus_6x2_plus_11x_minus_6.coeff(Nat.1) +
            Int.3 * (int_cubic_x3_minus_6x2_plus_11x_minus_6.coeff(Nat.2) +
                Int.3 * (int_cubic_x3_minus_6x2_plus_11x_minus_6.coeff(Nat.3) + Int.3 * Int.0)))
    cubic_coeff_zero(Int.1, -Int.6, Int.11, -Int.6)
    int_cubic_x3_minus_6x2_plus_11x_minus_6.coeff(Nat.0) = -Int.6
    cubic_coeff_one(Int.1, -Int.6, Int.11, -Int.6)
    int_cubic_x3_minus_6x2_plus_11x_minus_6.coeff(Nat.1) = Int.11
    cubic_coeff_two(Int.1, -Int.6, Int.11, -Int.6)
    int_cubic_x3_minus_6x2_plus_11x_minus_6.coeff(Nat.2) = -Int.6
    cubic_coeff_three(Int.1, -Int.6, Int.11, -Int.6)
    int_cubic_x3_minus_6x2_plus_11x_minus_6.coeff(Nat.3) = Int.1
    coeff_eval(int_cubic_x3_minus_6x2_plus_11x_minus_6.coeff, Int.3, Nat.4) =
        -Int.6 + Int.3 * (Int.11 + Int.3 * (-Int.6 + Int.3 * (Int.1 + Int.3 * Int.0)))
    -Int.6 + Int.3 * (Int.11 + Int.3 * (-Int.6 + Int.3 * (Int.1 + Int.3 * Int.0))) =
        -Int.6 + Int.3 * (Int.11 + Int.3 * (-Int.6 + Int.3 * Int.1))
    int_neg_six_add_three
    -Int.6 + Int.3 = -Int.3
    -Int.6 + Int.3 * (Int.11 + Int.3 * (-Int.6 + Int.3 * Int.1)) =
        -Int.6 + Int.3 * (Int.11 + Int.3 * -Int.3)
    int_three_mul_neg_three
    Int.3 * -Int.3 = -Int.9
    -Int.6 + Int.3 * (Int.11 + Int.3 * -Int.3) = -Int.6 + Int.3 * (Int.11 + -Int.9)
    int_eleven_add_neg_nine
    Int.11 + -Int.9 = Int.2
    -Int.6 + Int.3 * (Int.11 + -Int.9) = -Int.6 + Int.3 * Int.2
    int_three_mul_two
    Int.3 * Int.2 = Int.6
    -Int.6 + Int.3 * Int.2 = -Int.6 + Int.6
    add_neg(Int.6)
    Int.6 + -Int.6 = Int.0
    -Int.6 + Int.6 = Int.0
    -Int.6 + Int.3 * Int.2 = Int.0
    coeff_eval(int_cubic_x3_minus_6x2_plus_11x_minus_6.coeff, Int.3, Nat.4) = Int.0
    polynomial_eval(int_cubic_x3_minus_6x2_plus_11x_minus_6, Int.3) = Int.0
}

/// The integer cubic `X^3 - 6 X^2 + 11 X - 6` has the integer root `1`.
///
/// The value is computed in the Horner form at bound four.
theorem int_cubic_root_one {
    polynomial_eval(int_cubic_x3_minus_6x2_plus_11x_minus_6, Int.1) = Int.0
} by {
    cubic_support_bounded_by_four(Int.1, -Int.6, Int.11, -Int.6)
    polynomial_support_bounded_by(int_cubic_x3_minus_6x2_plus_11x_minus_6, Nat.4)
    polynomial_eval_eq_eval_bound_of_support_bounded(int_cubic_x3_minus_6x2_plus_11x_minus_6, Int.1, Nat.4)
    polynomial_eval(int_cubic_x3_minus_6x2_plus_11x_minus_6, Int.1) =
        polynomial_eval_bound(int_cubic_x3_minus_6x2_plus_11x_minus_6, Int.1, Nat.4)
    polynomial_eval_bound_eq_coeff_eval(int_cubic_x3_minus_6x2_plus_11x_minus_6, Int.1, Nat.4)
    polynomial_eval_bound(int_cubic_x3_minus_6x2_plus_11x_minus_6, Int.1, Nat.4) =
        coeff_eval(int_cubic_x3_minus_6x2_plus_11x_minus_6.coeff, Int.1, Nat.4)
    coeff_eval_four(int_cubic_x3_minus_6x2_plus_11x_minus_6.coeff, Int.1)
    coeff_eval(int_cubic_x3_minus_6x2_plus_11x_minus_6.coeff, Int.1, Nat.4) =
        int_cubic_x3_minus_6x2_plus_11x_minus_6.coeff(Nat.0) +
        Int.1 * (int_cubic_x3_minus_6x2_plus_11x_minus_6.coeff(Nat.1) +
            Int.1 * (int_cubic_x3_minus_6x2_plus_11x_minus_6.coeff(Nat.2) +
                Int.1 * (int_cubic_x3_minus_6x2_plus_11x_minus_6.coeff(Nat.3) + Int.1 * Int.0)))
    cubic_coeff_zero(Int.1, -Int.6, Int.11, -Int.6)
    int_cubic_x3_minus_6x2_plus_11x_minus_6.coeff(Nat.0) = -Int.6
    cubic_coeff_one(Int.1, -Int.6, Int.11, -Int.6)
    int_cubic_x3_minus_6x2_plus_11x_minus_6.coeff(Nat.1) = Int.11
    cubic_coeff_two(Int.1, -Int.6, Int.11, -Int.6)
    int_cubic_x3_minus_6x2_plus_11x_minus_6.coeff(Nat.2) = -Int.6
    cubic_coeff_three(Int.1, -Int.6, Int.11, -Int.6)
    int_cubic_x3_minus_6x2_plus_11x_minus_6.coeff(Nat.3) = Int.1
    coeff_eval(int_cubic_x3_minus_6x2_plus_11x_minus_6.coeff, Int.1, Nat.4) =
        -Int.6 + Int.1 * (Int.11 + Int.1 * (-Int.6 + Int.1 * (Int.1 + Int.1 * Int.0)))
    -Int.6 + Int.1 * (Int.11 + Int.1 * (-Int.6 + Int.1 * (Int.1 + Int.1 * Int.0))) =
        -Int.6 + Int.1 * (Int.11 + Int.1 * (-Int.6 + Int.1 * Int.1))
    -Int.6 + Int.1 * (Int.11 + Int.1 * (-Int.6 + Int.1 * Int.1)) =
        -Int.6 + Int.1 * (Int.11 + Int.1 * (-Int.6 + Int.1))
    int_neg_from_nat_add_neg(Nat.6, Nat.1, Nat.5)
    Nat.1 + Nat.5 = Nat.6
    -Int.from_nat(Nat.6) + Int.from_nat(Nat.1) = -Int.from_nat(Nat.5)
    Int.5 = Int.from_nat(Nat.5)
    Int.1 = Int.from_nat(Nat.1)
    -Int.6 + Int.1 = -Int.5
    -Int.6 + Int.1 * (Int.11 + Int.1 * (-Int.6 + Int.1)) =
        -Int.6 + Int.1 * (Int.11 + Int.1 * -Int.5)
    Int.1 * -Int.5 = -Int.5
    -Int.6 + Int.1 * (Int.11 + Int.1 * -Int.5) = -Int.6 + Int.1 * (Int.11 + -Int.5)
    int_neg_from_nat_add_cancel(Nat.5, Nat.11, Nat.6)
    Nat.6 = Nat.5.suc
    Nat.5 + Nat.6 = Nat.5 + Nat.5.suc
    add_suc_right(Nat.5, Nat.5)
    Nat.5 + Nat.5.suc = (Nat.5 + Nat.5).suc
    Nat.5 + Nat.5 = Nat.10
    (Nat.5 + Nat.5).suc = Nat.10.suc
    Nat.10.suc = Nat.11
    Nat.5 + Nat.6 = Nat.11
    -Int.from_nat(Nat.5) + Int.from_nat(Nat.11) = Int.from_nat(Nat.6)
    Int.11 = Int.from_nat(Nat.11)
    Int.6 = Int.from_nat(Nat.6)
    Int.11 + -Int.5 = Int.6
    -Int.6 + Int.1 * (Int.11 + -Int.5) = -Int.6 + Int.1 * Int.6
    Int.1 * Int.6 = Int.6
    -Int.6 + Int.1 * Int.6 = -Int.6 + Int.6
    add_neg(Int.6)
    Int.6 + -Int.6 = Int.0
    -Int.6 + Int.6 = Int.0
    -Int.6 + Int.1 * Int.6 = Int.0
    coeff_eval(int_cubic_x3_minus_6x2_plus_11x_minus_6.coeff, Int.1, Nat.4) = Int.0
    polynomial_eval(int_cubic_x3_minus_6x2_plus_11x_minus_6, Int.1) = Int.0
}

/// A rational fraction with denominator one is the numerator.
lemma rat_div_one(x: Rat) {
    x / Rat.1 = x
} by {
    x / Rat.1 = x * Rat.1.inverse
    inverse_one[Rat]
    Rat.1.inverse = Rat.1
    x / Rat.1 = x * Rat.1
    x * Rat.1 = x
    x / Rat.1 = x
}

/// If `a` divides `-b` then `a` divides `b`.
theorem int_divides_neg_of_divides(a: Int, b: Int) {
    a.divides(-b) implies a.divides(b)
} by {
    if a.divides(-b) {
        (a.divides(-b) = exists(d: Int) { d * a = -b })
        let (d: Int) satisfy {
            d * a = -b
        }
        mul_neg_left(d, a)
        -d * a = -(d * a)
        -(d * a) = -(-b)
        inverse_inverse[Int](b)
        -(-b) = b
        -d * a = b
        exists(d0: Int) { d0 * a = b }
        a.divides(b)
    }
}

/// An integer root of the integer cubic is a root of the rationalised cubic:
/// the root `1`.
theorem rat_cubic_x3_minus_6x2_plus_11x_minus_6_root_one {
    polynomial_eval(rat_cubic_polynomial(Int.1, -Int.6, Int.11, -Int.6), Rat.from_int(Int.1)) = Rat.0
} by {
    int_cubic_root_one
    polynomial_eval(int_cubic_x3_minus_6x2_plus_11x_minus_6, Int.1) = Int.0
    polynomial_eval_map(int_to_rat_hom, int_cubic_x3_minus_6x2_plus_11x_minus_6, Int.1)
    polynomial_eval(polynomial_map(int_to_rat_hom, int_cubic_x3_minus_6x2_plus_11x_minus_6),
        int_to_rat_hom.hom(Int.1)) =
        int_to_rat_hom.hom(polynomial_eval(int_cubic_x3_minus_6x2_plus_11x_minus_6, Int.1))
    int_to_rat_hom_apply(Int.1)
    int_to_rat_hom.hom(Int.1) = Rat.from_int(Int.1)
    int_to_rat_hom_apply(polynomial_eval(int_cubic_x3_minus_6x2_plus_11x_minus_6, Int.1))
    int_to_rat_hom.hom(polynomial_eval(int_cubic_x3_minus_6x2_plus_11x_minus_6, Int.1)) =
        Rat.from_int(polynomial_eval(int_cubic_x3_minus_6x2_plus_11x_minus_6, Int.1))
    Rat.from_int(Int.0) = Rat.0
    polynomial_eval(rat_cubic_polynomial(Int.1, -Int.6, Int.11, -Int.6), Rat.from_int(Int.1)) = Rat.0
}

/// An integer root of the integer cubic is a root of the rationalised cubic:
/// the root `2`.
theorem rat_cubic_x3_minus_6x2_plus_11x_minus_6_root_two {
    polynomial_eval(rat_cubic_polynomial(Int.1, -Int.6, Int.11, -Int.6), Rat.from_int(Int.2)) = Rat.0
} by {
    int_cubic_root_two
    polynomial_eval(int_cubic_x3_minus_6x2_plus_11x_minus_6, Int.2) = Int.0
    polynomial_eval_map(int_to_rat_hom, int_cubic_x3_minus_6x2_plus_11x_minus_6, Int.2)
    polynomial_eval(polynomial_map(int_to_rat_hom, int_cubic_x3_minus_6x2_plus_11x_minus_6),
        int_to_rat_hom.hom(Int.2)) =
        int_to_rat_hom.hom(polynomial_eval(int_cubic_x3_minus_6x2_plus_11x_minus_6, Int.2))
    int_to_rat_hom_apply(Int.2)
    int_to_rat_hom.hom(Int.2) = Rat.from_int(Int.2)
    int_to_rat_hom_apply(polynomial_eval(int_cubic_x3_minus_6x2_plus_11x_minus_6, Int.2))
    int_to_rat_hom.hom(polynomial_eval(int_cubic_x3_minus_6x2_plus_11x_minus_6, Int.2)) =
        Rat.from_int(polynomial_eval(int_cubic_x3_minus_6x2_plus_11x_minus_6, Int.2))
    Rat.from_int(Int.0) = Rat.0
    polynomial_eval(rat_cubic_polynomial(Int.1, -Int.6, Int.11, -Int.6), Rat.from_int(Int.2)) = Rat.0
}

/// An integer root of the integer cubic is a root of the rationalised cubic:
/// the root `3`.
theorem rat_cubic_x3_minus_6x2_plus_11x_minus_6_root_three {
    polynomial_eval(rat_cubic_polynomial(Int.1, -Int.6, Int.11, -Int.6), Rat.from_int(Int.3)) = Rat.0
} by {
    int_cubic_root_three
    polynomial_eval(int_cubic_x3_minus_6x2_plus_11x_minus_6, Int.3) = Int.0
    polynomial_eval_map(int_to_rat_hom, int_cubic_x3_minus_6x2_plus_11x_minus_6, Int.3)
    polynomial_eval(polynomial_map(int_to_rat_hom, int_cubic_x3_minus_6x2_plus_11x_minus_6),
        int_to_rat_hom.hom(Int.3)) =
        int_to_rat_hom.hom(polynomial_eval(int_cubic_x3_minus_6x2_plus_11x_minus_6, Int.3))
    int_to_rat_hom_apply(Int.3)
    int_to_rat_hom.hom(Int.3) = Rat.from_int(Int.3)
    int_to_rat_hom_apply(polynomial_eval(int_cubic_x3_minus_6x2_plus_11x_minus_6, Int.3))
    int_to_rat_hom.hom(polynomial_eval(int_cubic_x3_minus_6x2_plus_11x_minus_6, Int.3)) =
        Rat.from_int(polynomial_eval(int_cubic_x3_minus_6x2_plus_11x_minus_6, Int.3))
    Rat.from_int(Int.0) = Rat.0
    polynomial_eval(rat_cubic_polynomial(Int.1, -Int.6, Int.11, -Int.6), Rat.from_int(Int.3)) = Rat.0
}

/// The root `1` of `X^3 - 6 X^2 + 11 X - 6`: by the rational root theorem the
/// numerator `1` divides the constant term `-6`, so `1` divides `6`.
theorem int_cubic_x3_minus_6x2_plus_11x_minus_6_root_one_divides_six {
    Int.1.divides(Int.6)
} by {
    cubic_rational_root_numerator_divides(Int.1, -Int.6, Int.11, -Int.6, Int.1, Int.1)
    Int.1 != Int.0
    is_coprime_one_right(Int.1)
    is_coprime(Int.1, Int.1)
    rat_div_one(Rat.from_int(Int.1))
    Rat.from_int(Int.1) / Rat.1 = Rat.from_int(Int.1)
    Rat.from_int(Int.1) = Rat.1
    Rat.from_int(Int.1) / Rat.from_int(Int.1) = Rat.from_int(Int.1)
    rat_cubic_x3_minus_6x2_plus_11x_minus_6_root_one
    polynomial_eval(rat_cubic_polynomial(Int.1, -Int.6, Int.11, -Int.6), Rat.from_int(Int.1)) = Rat.0
    polynomial_eval(rat_cubic_polynomial(Int.1, -Int.6, Int.11, -Int.6),
        Rat.from_int(Int.1) / Rat.from_int(Int.1)) = Rat.0
    Int.1 != Int.0 and is_coprime(Int.1, Int.1) and
        polynomial_eval(rat_cubic_polynomial(Int.1, -Int.6, Int.11, -Int.6),
            Rat.from_int(Int.1) / Rat.from_int(Int.1)) = Rat.0
    Int.1.divides(-Int.6)
    int_divides_neg_of_divides(Int.1, Int.6)
    Int.1.divides(-Int.6) implies Int.1.divides(Int.6)
    Int.1.divides(Int.6)
}

/// The root `2` of `X^3 - 6 X^2 + 11 X - 6`: by the rational root theorem the
/// numerator `2` divides the constant term `-6`, so `2` divides `6`.
theorem int_cubic_x3_minus_6x2_plus_11x_minus_6_root_two_divides_six {
    Int.2.divides(Int.6)
} by {
    cubic_rational_root_numerator_divides(Int.1, -Int.6, Int.11, -Int.6, Int.2, Int.1)
    Int.1 != Int.0
    is_coprime_one_right(Int.2)
    is_coprime(Int.2, Int.1)
    rat_div_one(Rat.from_int(Int.2))
    Rat.from_int(Int.2) / Rat.1 = Rat.from_int(Int.2)
    Rat.from_int(Int.1) = Rat.1
    Rat.from_int(Int.2) / Rat.from_int(Int.1) = Rat.from_int(Int.2)
    rat_cubic_x3_minus_6x2_plus_11x_minus_6_root_two
    polynomial_eval(rat_cubic_polynomial(Int.1, -Int.6, Int.11, -Int.6), Rat.from_int(Int.2)) = Rat.0
    polynomial_eval(rat_cubic_polynomial(Int.1, -Int.6, Int.11, -Int.6),
        Rat.from_int(Int.2) / Rat.from_int(Int.1)) = Rat.0
    Int.1 != Int.0 and is_coprime(Int.2, Int.1) and
        polynomial_eval(rat_cubic_polynomial(Int.1, -Int.6, Int.11, -Int.6),
            Rat.from_int(Int.2) / Rat.from_int(Int.1)) = Rat.0
    Int.2.divides(-Int.6)
    int_divides_neg_of_divides(Int.2, Int.6)
    Int.2.divides(-Int.6) implies Int.2.divides(Int.6)
    Int.2.divides(Int.6)
}

/// The root `3` of `X^3 - 6 X^2 + 11 X - 6`: by the rational root theorem the
/// numerator `3` divides the constant term `-6`, so `3` divides `6`.
theorem int_cubic_x3_minus_6x2_plus_11x_minus_6_root_three_divides_six {
    Int.3.divides(Int.6)
} by {
    cubic_rational_root_numerator_divides(Int.1, -Int.6, Int.11, -Int.6, Int.3, Int.1)
    Int.1 != Int.0
    is_coprime_one_right(Int.3)
    is_coprime(Int.3, Int.1)
    rat_div_one(Rat.from_int(Int.3))
    Rat.from_int(Int.3) / Rat.1 = Rat.from_int(Int.3)
    Rat.from_int(Int.1) = Rat.1
    Rat.from_int(Int.3) / Rat.from_int(Int.1) = Rat.from_int(Int.3)
    rat_cubic_x3_minus_6x2_plus_11x_minus_6_root_three
    polynomial_eval(rat_cubic_polynomial(Int.1, -Int.6, Int.11, -Int.6), Rat.from_int(Int.3)) = Rat.0
    polynomial_eval(rat_cubic_polynomial(Int.1, -Int.6, Int.11, -Int.6),
        Rat.from_int(Int.3) / Rat.from_int(Int.1)) = Rat.0
    Int.1 != Int.0 and is_coprime(Int.3, Int.1) and
        polynomial_eval(rat_cubic_polynomial(Int.1, -Int.6, Int.11, -Int.6),
            Rat.from_int(Int.3) / Rat.from_int(Int.1)) = Rat.0
    Int.3.divides(-Int.6)
    int_divides_neg_of_divides(Int.3, Int.6)
    Int.3.divides(-Int.6) implies Int.3.divides(Int.6)
    Int.3.divides(Int.6)
}
