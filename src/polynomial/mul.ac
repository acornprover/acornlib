/// Core coefficientwise convolution for polynomial multiplication.

from finite_set import FiniteSet, fs_from_list
from algebra.module.finite_support import has_support_in, is_finitely_supported
from list import partial, partial_pointwise_eq, partial_split_last, partial_zero, range_contains_all_leq
from nat import Nat
from nat import add_sub, alt_induction, lt_add_left, lt_and_lte, lt_imp_lte_suc, lt_suc_right
from polynomial.add_monoid_algebra_bridge import coeff_zero_at, coeff_zero_from, coeff_zero_from_apply,
    coeff_zero_from_at
from polynomial.base import Polynomial
from polynomial.eval import polynomial_support_bounded_by, polynomial_support_bound_exists
from semiring import Semiring
from data.basic.set import list_set, list_set_contains_eq

numerals Nat

lemma polynomial_mul_coeff_zero_from_at_intro[R: Semiring](c: Nat -> R, n: Nat, k: Nat) {
    not k < n and c(k) = R.0 implies coeff_zero_from_at(c, n, k)
} by {
    if not k < n and c(k) = R.0 {
        coeff_zero_at(c, k)
        coeff_zero_from_at(c, n, k) = (not k < n implies coeff_zero_at(c, k))
        coeff_zero_from_at(c, n, k)
    }
}

lemma polynomial_mul_coeff_zero_from_intro[R: Semiring](c: Nat -> R, n: Nat) {
    (forall(k: Nat) { coeff_zero_from_at(c, n, k) }) implies coeff_zero_from(c, n)
} by {
    forall(k: Nat) {
        coeff_zero_from_at(c, n, k)
    }
    coeff_zero_from(c, n) = forall(k: Nat) {
        coeff_zero_from_at(c, n, k)
    }
}

/// The `i`th summand in the coefficient of `X^k` in a product.
define polynomial_mul_term_coeff[R: Semiring](p: Polynomial[R], q: Polynomial[R], k: Nat, i: Nat) -> R {
    p.coeff(i) * q.coeff(k - i)
}

/// The coefficient of `X^k` in the convolution product.
define polynomial_mul_coeff[R: Semiring](p: Polynomial[R], q: Polynomial[R], k: Nat) -> R {
    partial(function(i: Nat) { polynomial_mul_term_coeff(p, q, k, i) }, k.suc)
}

/// The zero natural-indexed coefficient function over a semiring.
define polynomial_zero_nat_function[R: Semiring](i: Nat) -> R {
    R.0
}

lemma polynomial_partial_zero_nat_function[R: Semiring](n: Nat) {
    partial(polynomial_zero_nat_function[R], n) = R.0
} by {
    define statement(k: Nat) -> Bool {
        partial(polynomial_zero_nat_function[R], k) = R.0
    }

    partial_zero[R](polynomial_zero_nat_function[R])
    statement(Nat.0)

    forall(k: Nat) {
        if statement(k) {
            partial_split_last[R](polynomial_zero_nat_function[R], k)
            partial(polynomial_zero_nat_function[R], k.suc) =
                partial(polynomial_zero_nat_function[R], k) + polynomial_zero_nat_function[R](k)
            polynomial_zero_nat_function[R](k) = R.0
            partial(polynomial_zero_nat_function[R], k) = R.0
            partial(polynomial_zero_nat_function[R], k.suc) = R.0 + R.0
            R.0 + R.0 = R.0
            partial(polynomial_zero_nat_function[R], k.suc) = R.0
            statement(k.suc)
        }
    }

    statement(Nat.0) and forall(k: Nat) {
        statement(k) implies statement(k.suc)
    }
    alt_induction(statement)
    forall(k: Nat) { statement(k) }
    statement(n)
}

lemma polynomial_partial_zero_of_pointwise_zero[R: Semiring](f: Nat -> R, n: Nat) {
    (forall(i: Nat) { i < n implies f(i) = R.0 }) implies partial(f, n) = R.0
} by {
    if forall(i: Nat) { i < n implies f(i) = R.0 } {
        forall(i: Nat) {
            if i < n {
                f(i) = R.0
                polynomial_zero_nat_function[R](i) = R.0
                f(i) = polynomial_zero_nat_function[R](i)
            }
        }
        partial_pointwise_eq[R](f, polynomial_zero_nat_function[R], n)
        partial(f, n) = partial(polynomial_zero_nat_function[R], n)
        polynomial_partial_zero_nat_function[R](n)
        partial(f, n) = R.0
    }
}

/// If both term indices are inside their bounds, the product coefficient index is inside the sum bound.
lemma polynomial_mul_index_lt_add_of_term_bounds(m: Nat, n: Nat, k: Nat, i: Nat) {
    i < k.suc and i < m and (k - i) < n implies k < m + n
} by {
    if i < k.suc and i < m and (k - i) < n {
        if i = k {
            k < m
            m + n = m + n
            m <= m + n
            lt_and_lte(k, m, m + n)
            k < m + n
        } else {
            lt_suc_right(i, k)
            i < k
            lt_imp_lte_suc(i, k)
            i.suc <= k
            i <= k
            add_sub(k, i)
            k - i + i = k
            lt_add_left(i, k - i, n)
            i + (k - i) < i + n
            i + (k - i) = k - i + i
            i + (k - i) = k
            k < i + n
            lt_add_left(n, i, m)
            n + i < n + m
            i + n = n + i
            n + m = m + n
            i + n < m + n
            lt_and_lte(k, i + n, m + n)
            k < m + n
        }
    }
}

lemma polynomial_mul_term_coeff_zero_of_left_support_bounded[R: Semiring](
    p: Polynomial[R],
    q: Polynomial[R],
    m: Nat,
    k: Nat,
    i: Nat
) {
    polynomial_support_bounded_by(p, m) and not i < m implies
    polynomial_mul_term_coeff(p, q, k, i) = R.0
} by {
    if polynomial_support_bounded_by(p, m) and not i < m {
        polynomial_support_bounded_by(p, m) = coeff_zero_from(p.coeff, m)
        coeff_zero_from(p.coeff, m)
        coeff_zero_from_apply(p.coeff, m, i)
        p.coeff(i) = R.0
        polynomial_mul_term_coeff(p, q, k, i) = p.coeff(i) * q.coeff(k - i)
        polynomial_mul_term_coeff(p, q, k, i) = R.0 * q.coeff(k - i)
        R.0 * q.coeff(k - i) = R.0
        polynomial_mul_term_coeff(p, q, k, i) = R.0
    }
}

lemma polynomial_mul_term_coeff_zero_of_right_support_bounded[R: Semiring](
    p: Polynomial[R],
    q: Polynomial[R],
    n: Nat,
    k: Nat,
    i: Nat
) {
    polynomial_support_bounded_by(q, n) and not k - i < n implies
    polynomial_mul_term_coeff(p, q, k, i) = R.0
} by {
    if polynomial_support_bounded_by(q, n) and not k - i < n {
        polynomial_support_bounded_by(q, n) = coeff_zero_from(q.coeff, n)
        coeff_zero_from(q.coeff, n)
        coeff_zero_from_apply(q.coeff, n, k - i)
        q.coeff(k - i) = R.0
        polynomial_mul_term_coeff(p, q, k, i) = p.coeff(i) * q.coeff(k - i)
        polynomial_mul_term_coeff(p, q, k, i) = p.coeff(i) * R.0
        p.coeff(i) * R.0 = R.0
        polynomial_mul_term_coeff(p, q, k, i) = R.0
    }
}

lemma polynomial_mul_term_coeff_zero_of_support_bounded[R: Semiring](
    p: Polynomial[R],
    q: Polynomial[R],
    m: Nat,
    n: Nat,
    k: Nat,
    i: Nat
) {
    polynomial_support_bounded_by(p, m) and polynomial_support_bounded_by(q, n) and
    not k < m + n and i < k.suc implies polynomial_mul_term_coeff(p, q, k, i) = R.0
} by {
    if polynomial_support_bounded_by(p, m) and polynomial_support_bounded_by(q, n) and
        not k < m + n and i < k.suc {
        if i < m {
            if k - i < n {
                polynomial_mul_index_lt_add_of_term_bounds(m, n, k, i)
                k < m + n
                false
            }
            not k - i < n
            polynomial_support_bounded_by(q, n) = coeff_zero_from(q.coeff, n)
            coeff_zero_from(q.coeff, n)
            coeff_zero_from_apply(q.coeff, n, k - i)
            q.coeff(k - i) = R.0
            polynomial_mul_term_coeff(p, q, k, i) = p.coeff(i) * q.coeff(k - i)
            polynomial_mul_term_coeff(p, q, k, i) = p.coeff(i) * R.0
            p.coeff(i) * R.0 = R.0
            polynomial_mul_term_coeff(p, q, k, i) = R.0
        } else {
            not i < m
            polynomial_support_bounded_by(p, m) = coeff_zero_from(p.coeff, m)
            coeff_zero_from(p.coeff, m)
            coeff_zero_from_apply(p.coeff, m, i)
            p.coeff(i) = R.0
            polynomial_mul_term_coeff(p, q, k, i) = p.coeff(i) * q.coeff(k - i)
            polynomial_mul_term_coeff(p, q, k, i) = R.0 * q.coeff(k - i)
            R.0 * q.coeff(k - i) = R.0
            polynomial_mul_term_coeff(p, q, k, i) = R.0
        }
    }
}

/// If both factors are support-bounded and `k` is outside the sum bound, the convolution coefficient vanishes.
theorem polynomial_mul_coeff_zero_of_support_bounded[R: Semiring](
    p: Polynomial[R],
    q: Polynomial[R],
    m: Nat,
    n: Nat,
    k: Nat
) {
    polynomial_support_bounded_by(p, m) and polynomial_support_bounded_by(q, n) and
    not k < m + n implies polynomial_mul_coeff(p, q, k) = R.0
} by {
    if polynomial_support_bounded_by(p, m) and polynomial_support_bounded_by(q, n) and
        not k < m + n {
        let term_fn: Nat -> R = function(i: Nat) {
            polynomial_mul_term_coeff(p, q, k, i)
        }
        forall(i: Nat) {
            if i < k.suc {
                polynomial_mul_term_coeff_zero_of_support_bounded(p, q, m, n, k, i)
                polynomial_mul_term_coeff(p, q, k, i) = R.0
                term_fn(i) = polynomial_mul_term_coeff(p, q, k, i)
                term_fn(i) = R.0
            }
        }
        polynomial_partial_zero_of_pointwise_zero[R](term_fn, k.suc)
        partial(term_fn, k.suc) = R.0
        polynomial_mul_coeff(p, q, k) = partial(term_fn, k.suc)
        polynomial_mul_coeff(p, q, k) = R.0
    }
}

/// The convolution coefficient function is finitely supported.
theorem polynomial_mul_coeff_is_finitely_supported[R: Semiring](p: Polynomial[R], q: Polynomial[R]) {
    is_finitely_supported(polynomial_mul_coeff(p, q))
} by {
    polynomial_support_bound_exists(p)
    polynomial_support_bound_exists(q)
    let m: Nat satisfy {
        polynomial_support_bounded_by(p, m)
    }
    let n: Nat satisfy {
        polynomial_support_bounded_by(q, n)
    }
    let bound = m + n
    let s = fs_from_list[Nat](bound.range)
    forall(k: Nat) {
        if not s.contains(k) {
            if k < bound {
                range_contains_all_leq(bound)
                bound.range.contains(k)
                s = fs_from_list[Nat](bound.range)
                fs_from_list[Nat](bound.range).underlying_set = list_set(bound.range)
                fs_from_list[Nat](bound.range).contains(k) =
                    fs_from_list[Nat](bound.range).underlying_set.contains(k)
                fs_from_list[Nat](bound.range).underlying_set.contains(k) = list_set(bound.range).contains(k)
                list_set_contains_eq(bound.range, k)
                list_set(bound.range).contains(k) = bound.range.contains(k)
                fs_from_list[Nat](bound.range).contains(k)
                s.contains(k)
                false
            }
            not k < bound
            bound = m + n
            not k < m + n
            polynomial_mul_coeff_zero_of_support_bounded(p, q, m, n, k)
            polynomial_mul_coeff(p, q, k) = R.0
        }
    }
    has_support_in(polynomial_mul_coeff(p, q), s) = forall(k: Nat) {
        not s.contains(k) implies polynomial_mul_coeff(p, q, k) = R.0
    }
    has_support_in(polynomial_mul_coeff(p, q), s)
    exists(u: FiniteSet[Nat]) {
        u = s and has_support_in(polynomial_mul_coeff(p, q), u)
    }
}

/// The bundled polynomial product with convolution coefficients.
let polynomial_mul[R: Semiring](p: Polynomial[R], q: Polynomial[R]) -> result: Polynomial[R] satisfy {
    Polynomial[R].new(polynomial_mul_coeff(p, q)) = Option.some(result)
} by {
    polynomial_mul_coeff_is_finitely_supported(p, q)
}

/// The bundled product has the convolution coefficient function.
theorem polynomial_mul_coeff_eq[R: Semiring](p: Polynomial[R], q: Polynomial[R]) {
    polynomial_mul(p, q).coeff = polynomial_mul_coeff(p, q)
} by {
    Polynomial[R].new(polynomial_mul_coeff(p, q)) = Option.some(polynomial_mul(p, q))
    polynomial_mul(p, q).coeff = polynomial_mul_coeff(p, q)
}

/// The `k`th coefficient of the bundled product is the convolution coefficient.
theorem polynomial_mul_coeff_apply[R: Semiring](p: Polynomial[R], q: Polynomial[R], k: Nat) {
    polynomial_mul(p, q).coeff(k) = polynomial_mul_coeff(p, q, k)
} by {
    polynomial_mul_coeff_eq(p, q)
    polynomial_mul(p, q).coeff(k) = polynomial_mul_coeff(p, q, k)
}

/// The product of polynomials supported below `m` and `n` is supported below `m + n`.
theorem polynomial_mul_support_bounded_by_add[R: Semiring](
    p: Polynomial[R],
    q: Polynomial[R],
    m: Nat,
    n: Nat
) {
    polynomial_support_bounded_by(p, m) and polynomial_support_bounded_by(q, n) implies
    polynomial_support_bounded_by(polynomial_mul(p, q), m + n)
} by {
    if polynomial_support_bounded_by(p, m) and polynomial_support_bounded_by(q, n) {
        forall(k: Nat) {
            if k < m + n {
                coeff_zero_from_at(polynomial_mul(p, q).coeff, m + n, k) =
                    (not k < m + n implies coeff_zero_at(polynomial_mul(p, q).coeff, k))
                coeff_zero_from_at(polynomial_mul(p, q).coeff, m + n, k)
            } else {
                not k < m + n
                polynomial_mul_coeff_apply(p, q, k)
                polynomial_mul_coeff_zero_of_support_bounded(p, q, m, n, k)
                polynomial_mul(p, q).coeff(k) = R.0
                coeff_zero_at(polynomial_mul(p, q).coeff, k)
                polynomial_mul_coeff_zero_from_at_intro(polynomial_mul(p, q).coeff, m + n, k)
                coeff_zero_from_at(polynomial_mul(p, q).coeff, m + n, k)
            }
        }
        polynomial_mul_coeff_zero_from_intro(polynomial_mul(p, q).coeff, m + n)
        coeff_zero_from(polynomial_mul(p, q).coeff, m + n)
        polynomial_support_bounded_by(polynomial_mul(p, q), m + n)
    }
}
