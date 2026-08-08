/// Bounded evaluation for finitely supported univariate polynomials.

from finite_set import FiniteSet, fs_from_list, finite_set_has_list
from list import List, is_upper_bound, list_max, list_max_is_upper_bound, range_contains_all_leq,
    range_does_not_contain_geq, upper_bound_contains
from algebra.module.finite_support import has_support_in, has_support_in_apply_zero, is_finitely_supported
from data.basic.function_algebra import pointwise_add
from data.basic.functions import function_extensionality
from nat import Nat, alt_induction, lt_and_lte, lt_imp_lte_suc, lt_or_lte, lt_suc,
    lte_and_lt, not_lt_zero
from polynomial.add_monoid_algebra_bridge import coeff_eval_eq_of_zero_from_le,
    coeff_zero_at, coeff_zero_from, coeff_zero_from_apply, coeff_zero_from_at,
    finite_set_nat_bounded_by, finite_set_nat_bounded_by_not_contains,
    finite_set_nat_outside_bound
from polynomial.base import Polynomial, polynomial_add_coeff, polynomial_constant_coeff_of_ne_zero,
    polynomial_constant_coeff_zero, polynomial_one_coeff_of_ne_zero, polynomial_one_coeff_zero,
    polynomial_zero_coeff
from polynomial.coeff_eval import coeff_eval, coeff_eval_add, coeff_mul_x_minus_a,
    coeff_quotient, coeff_remainder_theorem, coeff_tail
from semiring import Semiring
from data.basic.set import list_set, list_set_contains_eq
from comm_ring import CommRing

/// A polynomial has no nonzero coefficient at or above `n`.
define polynomial_support_bounded_by[R: Semiring](p: Polynomial[R], n: Nat) -> Bool {
    coeff_zero_from(p.coeff, n)
}

/// Bounded Horner evaluation of a polynomial using its first `n` coefficients.
define polynomial_eval_bound[R: Semiring](p: Polynomial[R], x: R, n: Nat) -> R {
    coeff_eval(p.coeff, x, n)
}

/// Bounded polynomial evaluation unfolds to coefficient evaluation on the coefficient function.
theorem polynomial_eval_bound_eq_coeff_eval[R: Semiring](p: Polynomial[R], x: R, n: Nat) {
    polynomial_eval_bound(p, x, n) = coeff_eval(p.coeff, x, n)
}

/// The synthetic quotient coefficients for division by `X - a`, at a fixed bound.
define polynomial_quotient_coeff[R: Semiring](p: Polynomial[R], a: R, n: Nat, i: Nat) -> R {
    coeff_quotient(p.coeff, a, n, i)
}

/// Bounded evaluation of the synthetic quotient associated to a polynomial.
define polynomial_quotient_eval_bound[R: Semiring](p: Polynomial[R], a: R, x: R, n: Nat) -> R {
    coeff_eval(polynomial_quotient_coeff(p, a, n), x, n)
}

/// A coefficient function witnesses bounded divisibility by the linear factor `X - a`.
define polynomial_x_minus_a_witness_bound[R: CommRing](
    p: Polynomial[R],
    a: R,
    n: Nat,
    q: Nat -> R
) -> Bool {
    forall(x: R) {
        polynomial_eval_bound(p, x, n) = coeff_mul_x_minus_a(q, a, x, n)
    }
}

/// Bounded divisibility by the linear factor `X - a` at a fixed evaluation bound.
define polynomial_divisible_by_x_minus_a_bound[R: CommRing](p: Polynomial[R], a: R, n: Nat) -> Bool {
    exists(q: Nat -> R) {
        polynomial_x_minus_a_witness_bound(p, a, n, q)
    }
}

lemma polynomial_nat_lte_add_right(a: Nat, b: Nat) {
    a <= a + b
} by {
    a + b = a + b
}

lemma polynomial_coeff_zero_from_at_intro[R: Semiring](c: Nat -> R, n: Nat, k: Nat) {
    not k < n and c(k) = R.0 implies coeff_zero_from_at(c, n, k)
} by {
    if not k < n and c(k) = R.0 {
        coeff_zero_at(c, k)
        coeff_zero_from_at(c, n, k) = (not k < n implies coeff_zero_at(c, k))
        coeff_zero_from_at(c, n, k)
    }
}

lemma polynomial_coeff_zero_from_intro[R: Semiring](c: Nat -> R, n: Nat) {
    (forall(k: Nat) { coeff_zero_from_at(c, n, k) }) implies coeff_zero_from(c, n)
} by {
    forall(k: Nat) {
        coeff_zero_from_at(c, n, k)
    }
    coeff_zero_from(c, n) = forall(k: Nat) {
        coeff_zero_from_at(c, n, k)
    }
}

lemma polynomial_finite_set_nat_bounded_by_intro(s: FiniteSet[Nat], n: Nat) {
    (forall(k: Nat) { finite_set_nat_outside_bound(s, n, k) }) implies finite_set_nat_bounded_by(s, n)
} by {
    forall(k: Nat) {
        finite_set_nat_outside_bound(s, n, k)
    }
    finite_set_nat_bounded_by(s, n) = forall(k: Nat) {
        finite_set_nat_outside_bound(s, n, k)
    }
}

lemma polynomial_range_support_bounded_by(n: Nat) {
    finite_set_nat_bounded_by(fs_from_list[Nat](n.range), n)
} by {
    forall(k: Nat) {
        if k < n {
            finite_set_nat_outside_bound(fs_from_list[Nat](n.range), n, k) =
                (not k < n implies not fs_from_list[Nat](n.range).contains(k))
            finite_set_nat_outside_bound(fs_from_list[Nat](n.range), n, k)
        } else {
            not k < n
            n <= k
            range_does_not_contain_geq(n, k)
            not n.range.contains(k)
            fs_from_list[Nat](n.range).underlying_set = list_set(n.range)
            fs_from_list[Nat](n.range).contains(k) = fs_from_list[Nat](n.range).underlying_set.contains(k)
            fs_from_list[Nat](n.range).underlying_set.contains(k) = list_set(n.range).contains(k)
            list_set_contains_eq(n.range, k)
            list_set(n.range).contains(k) = n.range.contains(k)
            not fs_from_list[Nat](n.range).contains(k)
            finite_set_nat_outside_bound(fs_from_list[Nat](n.range), n, k) =
                (not k < n implies not fs_from_list[Nat](n.range).contains(k))
            finite_set_nat_outside_bound(fs_from_list[Nat](n.range), n, k)
        }
    }
    polynomial_finite_set_nat_bounded_by_intro(fs_from_list[Nat](n.range), n)
}

/// The implication that one synthetic quotient coefficient outside the bound is zero.
define coeff_quotient_zero_from_bound_at[R: Semiring](c: Nat -> R, a: R, n: Nat, k: Nat) -> Bool {
    not k < n implies coeff_quotient(c, a, n, k) = R.0
}

/// The assertion that every synthetic quotient coefficient outside the bound is zero.
define coeff_quotient_zero_from_bound_property[R: Semiring](n: Nat) -> Bool {
    forall(c: Nat -> R, a: R, k: Nat) {
        coeff_quotient_zero_from_bound_at(c, a, n, k)
    }
}

lemma coeff_quotient_zero_from_bound_property_intro[R: Semiring](n: Nat) {
    (forall(c: Nat -> R, a: R, k: Nat) {
        coeff_quotient_zero_from_bound_at(c, a, n, k)
    }) implies coeff_quotient_zero_from_bound_property[R](n)
} by {
    forall(c: Nat -> R, a: R, k: Nat) {
        coeff_quotient_zero_from_bound_at(c, a, n, k)
    }
    coeff_quotient_zero_from_bound_property[R](n) = forall(c: Nat -> R, a: R, k: Nat) {
        coeff_quotient_zero_from_bound_at(c, a, n, k)
    }
}

lemma coeff_quotient_zero_from_bound_property_apply[R: Semiring](
    n: Nat,
    c: Nat -> R,
    a: R,
    k: Nat
) {
    coeff_quotient_zero_from_bound_property[R](n) implies
    coeff_quotient_zero_from_bound_at(c, a, n, k)
} by {
    if coeff_quotient_zero_from_bound_property[R](n) {
        coeff_quotient_zero_from_bound_property[R](n) = forall(d: Nat -> R, b: R, i: Nat) {
            coeff_quotient_zero_from_bound_at(d, b, n, i)
        }
        coeff_quotient_zero_from_bound_at(c, a, n, k)
    }
}

lemma coeff_quotient_zero_from_bound_property_all[R: Semiring](n: Nat) {
    coeff_quotient_zero_from_bound_property[R](n)
} by {
    define statement(bound: Nat) -> Bool {
        coeff_quotient_zero_from_bound_property[R](bound)
    }

    forall(c: Nat -> R, a: R, k: Nat) {
        if not k < Nat.0 {
            coeff_quotient(c, a, Nat.0, k) = R.0
        }
        coeff_quotient_zero_from_bound_at(c, a, Nat.0, k) =
            (not k < Nat.0 implies coeff_quotient(c, a, Nat.0, k) = R.0)
        coeff_quotient_zero_from_bound_at(c, a, Nat.0, k)
    }
    coeff_quotient_zero_from_bound_property_intro[R](Nat.0)
    coeff_quotient_zero_from_bound_property[R](Nat.0)
    statement(Nat.0)

    forall(bound: Nat) {
        if statement(bound) {
            forall(c: Nat -> R, a: R, k: Nat) {
                if not k < bound.suc {
                    if k = Nat.0 {
                        Nat.0 <= bound
                        lt_suc(bound)
                        bound < bound.suc
                        lte_and_lt(Nat.0, bound, bound.suc)
                        Nat.0 < bound.suc
                        k < bound.suc
                        false
                    }
                    let kp: Nat satisfy {
                        k = kp.suc
                    }
                    kp.suc = k
                    if kp < bound {
                        lt_imp_lte_suc(kp, bound)
                        kp.suc <= bound
                        lt_suc(bound)
                        bound < bound.suc
                        lte_and_lt(kp.suc, bound, bound.suc)
                        kp.suc < bound.suc
                        k < bound.suc
                        false
                    }
                    not kp < bound
                    statement(bound) = coeff_quotient_zero_from_bound_property[R](bound)
                    coeff_quotient_zero_from_bound_property_apply[R](bound, coeff_tail(c), a, kp)
                    coeff_quotient_zero_from_bound_at(coeff_tail(c), a, bound, kp)
                    coeff_quotient_zero_from_bound_at(coeff_tail(c), a, bound, kp) =
                        (not kp < bound implies coeff_quotient(coeff_tail(c), a, bound, kp) = R.0)
                    coeff_quotient(coeff_tail(c), a, bound, kp) = R.0
                    coeff_quotient(c, a, bound.suc, k) = coeff_quotient(coeff_tail(c), a, bound, kp)
                    coeff_quotient(c, a, bound.suc, k) = R.0
                }
                coeff_quotient_zero_from_bound_at(c, a, bound.suc, k) =
                    (not k < bound.suc implies coeff_quotient(c, a, bound.suc, k) = R.0)
                coeff_quotient_zero_from_bound_at(c, a, bound.suc, k)
            }
            coeff_quotient_zero_from_bound_property_intro[R](bound.suc)
            coeff_quotient_zero_from_bound_property[R](bound.suc)
            statement(bound.suc)
        }
    }

    statement(Nat.0) and forall(bound: Nat) {
        statement(bound) implies statement(bound.suc)
    }
    alt_induction(statement)
    forall(bound: Nat) { statement(bound) }
    statement(n)
}

lemma coeff_quotient_zero_of_not_lt_bound[R: Semiring](c: Nat -> R, a: R, n: Nat, k: Nat) {
    not k < n implies coeff_quotient(c, a, n, k) = R.0
} by {
    if not k < n {
        coeff_quotient_zero_from_bound_property_all[R](n)
        coeff_quotient_zero_from_bound_property_apply[R](n, c, a, k)
        coeff_quotient_zero_from_bound_at(c, a, n, k)
        coeff_quotient_zero_from_bound_at(c, a, n, k) =
            (not k < n implies coeff_quotient(c, a, n, k) = R.0)
        coeff_quotient(c, a, n, k) = R.0
    }
}

/// A synthetic quotient coefficient function is supported below its explicit bound.
theorem polynomial_quotient_coeff_has_support_in_range[R: Semiring](p: Polynomial[R], a: R, n: Nat) {
    has_support_in(polynomial_quotient_coeff(p, a, n), fs_from_list[Nat](n.range))
} by {
    forall(k: Nat) {
        if not fs_from_list[Nat](n.range).contains(k) {
            polynomial_range_support_bounded_by(n)
            finite_set_nat_bounded_by_not_contains(fs_from_list[Nat](n.range), n, k)
            if k < n {
                range_contains_all_leq(n)
                n.range.contains(k)
                fs_from_list[Nat](n.range).underlying_set = list_set(n.range)
                fs_from_list[Nat](n.range).contains(k) =
                    fs_from_list[Nat](n.range).underlying_set.contains(k)
                fs_from_list[Nat](n.range).underlying_set.contains(k) = list_set(n.range).contains(k)
                list_set_contains_eq(n.range, k)
                list_set(n.range).contains(k) = n.range.contains(k)
                fs_from_list[Nat](n.range).contains(k)
                false
            }
            not k < n
            coeff_quotient_zero_of_not_lt_bound(p.coeff, a, n, k)
            polynomial_quotient_coeff(p, a, n, k) = coeff_quotient(p.coeff, a, n, k)
            polynomial_quotient_coeff(p, a, n, k) = R.0
        }
    }
    let q = polynomial_quotient_coeff(p, a, n)
    let s = fs_from_list[Nat](n.range)
    q = polynomial_quotient_coeff(p, a, n)
    s = fs_from_list[Nat](n.range)
    forall(k: Nat) {
        if s.contains(k) {
            s.contains(k) or q(k) = R.0
        } else {
            not s.contains(k)
            not fs_from_list[Nat](n.range).contains(k)
            if k < n {
                range_contains_all_leq(n)
                n.range.contains(k)
                fs_from_list[Nat](n.range).underlying_set = list_set(n.range)
                fs_from_list[Nat](n.range).contains(k) =
                    fs_from_list[Nat](n.range).underlying_set.contains(k)
                fs_from_list[Nat](n.range).underlying_set.contains(k) = list_set(n.range).contains(k)
                list_set_contains_eq(n.range, k)
                list_set(n.range).contains(k) = n.range.contains(k)
                fs_from_list[Nat](n.range).contains(k)
                false
            }
            not k < n
            coeff_quotient_zero_of_not_lt_bound(p.coeff, a, n, k)
            q(k) = polynomial_quotient_coeff(p, a, n, k)
            polynomial_quotient_coeff(p, a, n, k) = coeff_quotient(p.coeff, a, n, k)
            q(k) = R.0
            s.contains(k) or q(k) = R.0
        }
    }
    has_support_in(q, s) = forall(k: Nat) {
        s.contains(k) or q(k) = R.0
    }
    has_support_in(q, s)
    has_support_in(polynomial_quotient_coeff(p, a, n), fs_from_list[Nat](n.range))
}

/// A synthetic quotient coefficient function is finitely supported.
theorem polynomial_quotient_coeff_is_finitely_supported[R: Semiring](p: Polynomial[R], a: R, n: Nat) {
    is_finitely_supported(polynomial_quotient_coeff(p, a, n))
} by {
    polynomial_quotient_coeff_has_support_in_range(p, a, n)
    exists(s: FiniteSet[Nat]) {
        s = fs_from_list[Nat](n.range) and has_support_in(polynomial_quotient_coeff(p, a, n), s)
    }
}

/// The synthetic quotient polynomial for division by `X - a`, at a fixed bound.
let polynomial_quotient[R: Semiring](p: Polynomial[R], a: R, n: Nat) -> result: Polynomial[R] satisfy {
    Polynomial[R].new(polynomial_quotient_coeff(p, a, n)) = Option.some(result)
} by {
    polynomial_quotient_coeff_is_finitely_supported(p, a, n)
}

/// The bundled synthetic quotient has the synthetic quotient coefficients.
theorem polynomial_quotient_coeff_eq[R: Semiring](p: Polynomial[R], a: R, n: Nat) {
    polynomial_quotient(p, a, n).coeff = polynomial_quotient_coeff(p, a, n)
} by {
    Polynomial[R].new(polynomial_quotient_coeff(p, a, n)) = Option.some(polynomial_quotient(p, a, n))
    polynomial_quotient(p, a, n).coeff = polynomial_quotient_coeff(p, a, n)
}

/// One coefficient of the bundled synthetic quotient is the corresponding synthetic quotient coefficient.
theorem polynomial_quotient_coeff_at[R: Semiring](p: Polynomial[R], a: R, n: Nat, k: Nat) {
    polynomial_quotient(p, a, n).coeff(k) = polynomial_quotient_coeff(p, a, n, k)
} by {
    polynomial_quotient_coeff_eq(p, a, n)
    polynomial_quotient(p, a, n).coeff(k) = polynomial_quotient_coeff(p, a, n, k)
}

/// Bounded evaluation of the bundled synthetic quotient matches quotient evaluation.
theorem polynomial_quotient_eval_bound_eq[R: Semiring](p: Polynomial[R], a: R, x: R, n: Nat) {
    polynomial_eval_bound(polynomial_quotient(p, a, n), x, n) = polynomial_quotient_eval_bound(p, a, x, n)
} by {
    polynomial_quotient_coeff_eq(p, a, n)
    polynomial_eval_bound(polynomial_quotient(p, a, n), x, n) =
        coeff_eval(polynomial_quotient(p, a, n).coeff, x, n)
    polynomial_quotient_eval_bound(p, a, x, n) = coeff_eval(polynomial_quotient_coeff(p, a, n), x, n)
    polynomial_eval_bound(polynomial_quotient(p, a, n), x, n) = polynomial_quotient_eval_bound(p, a, x, n)
}

/// A polynomial witnesses bounded divisibility by the linear factor `X - a`.
define polynomial_x_minus_a_polynomial_witness_bound[R: CommRing](
    p: Polynomial[R],
    a: R,
    n: Nat,
    q: Polynomial[R]
) -> Bool {
    forall(x: R) {
        polynomial_eval_bound(p, x, n) = polynomial_eval_bound(q, x, n) * (x - a)
    }
}

/// Applying a polynomial quotient witness at one evaluation point.
theorem polynomial_x_minus_a_polynomial_witness_bound_apply[R: CommRing](
    p: Polynomial[R],
    a: R,
    n: Nat,
    q: Polynomial[R],
    x: R
) {
    polynomial_x_minus_a_polynomial_witness_bound(p, a, n, q) implies
    polynomial_eval_bound(p, x, n) = polynomial_eval_bound(q, x, n) * (x - a)
} by {
    if polynomial_x_minus_a_polynomial_witness_bound(p, a, n, q) {
        polynomial_x_minus_a_polynomial_witness_bound(p, a, n, q) = forall(y: R) {
            polynomial_eval_bound(p, y, n) = polynomial_eval_bound(q, y, n) * (y - a)
        }
        polynomial_eval_bound(p, x, n) = polynomial_eval_bound(q, x, n) * (x - a)
    }
}

/// Bounded divisibility by `X - a` with a polynomial quotient witness.
define polynomial_divisible_by_x_minus_a_polynomial_bound[R: CommRing](p: Polynomial[R], a: R, n: Nat) -> Bool {
    exists(q: Polynomial[R]) {
        polynomial_x_minus_a_polynomial_witness_bound(p, a, n, q)
    }
}

/// A finite support contained below `n` gives a polynomial support bound at `n`.
theorem polynomial_support_bounded_by_of_has_support_in[R: Semiring](
    p: Polynomial[R],
    s: FiniteSet[Nat],
    n: Nat
) {
    has_support_in(p.coeff, s) and finite_set_nat_bounded_by(s, n) implies
    polynomial_support_bounded_by(p, n)
} by {
    if has_support_in(p.coeff, s) and finite_set_nat_bounded_by(s, n) {
        forall(k: Nat) {
            if k < n {
                coeff_zero_from_at(p.coeff, n, k) = (not k < n implies coeff_zero_at(p.coeff, k))
                coeff_zero_from_at(p.coeff, n, k)
            } else {
                not k < n
                finite_set_nat_bounded_by_not_contains(s, n, k)
                not s.contains(k)
                has_support_in_apply_zero[Nat, R](p.coeff, s, k)
                p.coeff(k) = R.0
                coeff_zero_at(p.coeff, k)
                polynomial_coeff_zero_from_at_intro(p.coeff, n, k)
                coeff_zero_from_at(p.coeff, n, k)
            }
        }
        polynomial_coeff_zero_from_intro(p.coeff, n)
        coeff_zero_from(p.coeff, n)
        polynomial_support_bounded_by(p, n)
    }
}

/// The bundled synthetic quotient has support bounded by its explicit bound.
theorem polynomial_quotient_support_bounded_by[R: Semiring](p: Polynomial[R], a: R, n: Nat) {
    polynomial_support_bounded_by(polynomial_quotient(p, a, n), n)
} by {
    polynomial_quotient_coeff_eq(p, a, n)
    polynomial_quotient_coeff_has_support_in_range(p, a, n)
    has_support_in(polynomial_quotient_coeff(p, a, n), fs_from_list[Nat](n.range))
    has_support_in(polynomial_quotient(p, a, n).coeff, fs_from_list[Nat](n.range))
    polynomial_range_support_bounded_by(n)
    finite_set_nat_bounded_by(fs_from_list[Nat](n.range), n)
    polynomial_support_bounded_by_of_has_support_in(polynomial_quotient(p, a, n), fs_from_list[Nat](n.range), n)
    polynomial_support_bounded_by(polynomial_quotient(p, a, n), n)
}

/// Bounded polynomial evaluation is independent of increasing a valid support bound.
theorem polynomial_eval_bound_eq_of_support_bounded_le[R: Semiring](
    p: Polynomial[R],
    x: R,
    n: Nat,
    m: Nat
) {
    polynomial_support_bounded_by(p, n) and n <= m implies
    polynomial_eval_bound(p, x, m) = polynomial_eval_bound(p, x, n)
} by {
    if polynomial_support_bounded_by(p, n) and n <= m {
        coeff_eval_eq_of_zero_from_le(p.coeff, x, n, m)
        coeff_eval(p.coeff, x, m) = coeff_eval(p.coeff, x, n)
        polynomial_eval_bound(p, x, m) = coeff_eval(p.coeff, x, m)
        polynomial_eval_bound(p, x, n) = coeff_eval(p.coeff, x, n)
        polynomial_eval_bound(p, x, m) = polynomial_eval_bound(p, x, n)
    }
}

/// A concrete finite support bound gives independence after enlarging the evaluation bound.
theorem polynomial_eval_bound_eq_of_has_support_in_le[R: Semiring](
    p: Polynomial[R],
    s: FiniteSet[Nat],
    x: R,
    n: Nat,
    m: Nat
) {
    has_support_in(p.coeff, s) and finite_set_nat_bounded_by(s, n) and n <= m implies
    polynomial_eval_bound(p, x, m) = polynomial_eval_bound(p, x, n)
} by {
    if has_support_in(p.coeff, s) and finite_set_nat_bounded_by(s, n) and n <= m {
        polynomial_support_bounded_by_of_has_support_in(p, s, n)
        polynomial_support_bounded_by(p, n)
        polynomial_eval_bound_eq_of_support_bounded_le(p, x, n, m)
        polynomial_eval_bound(p, x, m) = polynomial_eval_bound(p, x, n)
    }
}

/// Applying a polynomial support bound at an index outside the bound gives a zero coefficient.
theorem polynomial_support_bounded_by_apply[R: Semiring](p: Polynomial[R], n: Nat, k: Nat) {
    polynomial_support_bounded_by(p, n) and not k < n implies p.coeff(k) = R.0
} by {
    if polynomial_support_bounded_by(p, n) and not k < n {
        coeff_zero_from_apply(p.coeff, n, k)
        p.coeff(k) = R.0
    }
}

/// Enlarging a polynomial support bound preserves boundedness.
theorem polynomial_support_bounded_by_monotone[R: Semiring](
    p: Polynomial[R],
    n: Nat,
    m: Nat
) {
    polynomial_support_bounded_by(p, n) and n <= m implies polynomial_support_bounded_by(p, m)
} by {
    if polynomial_support_bounded_by(p, n) and n <= m {
        forall(k: Nat) {
            if k < m {
                coeff_zero_from_at(p.coeff, m, k) = (not k < m implies coeff_zero_at(p.coeff, k))
                coeff_zero_from_at(p.coeff, m, k)
            } else {
                not k < m
                if k < n {
                    lt_and_lte(k, n, m)
                    k < m
                    false
                }
                not k < n
                polynomial_support_bounded_by_apply(p, n, k)
                p.coeff(k) = R.0
                coeff_zero_at(p.coeff, k)
                polynomial_coeff_zero_from_at_intro(p.coeff, m, k)
                coeff_zero_from_at(p.coeff, m, k)
            }
        }
        polynomial_coeff_zero_from_intro(p.coeff, m)
        coeff_zero_from(p.coeff, m)
        polynomial_support_bounded_by(p, m)
    }
}

/// Bounded polynomial evaluation is independent of any chosen valid support bound.
theorem polynomial_eval_bound_eq_of_support_bounded[R: Semiring](
    p: Polynomial[R],
    x: R,
    n: Nat,
    m: Nat
) {
    polynomial_support_bounded_by(p, n) and polynomial_support_bounded_by(p, m) implies
    polynomial_eval_bound(p, x, n) = polynomial_eval_bound(p, x, m)
} by {
    if polynomial_support_bounded_by(p, n) and polynomial_support_bounded_by(p, m) {
        if n <= m {
            polynomial_eval_bound_eq_of_support_bounded_le(p, x, n, m)
            polynomial_eval_bound(p, x, m) = polynomial_eval_bound(p, x, n)
        } else {
            lt_or_lte(n, m)
            m <= n
            polynomial_eval_bound_eq_of_support_bounded_le(p, x, m, n)
            polynomial_eval_bound(p, x, n) = polynomial_eval_bound(p, x, m)
        }
    }
}

lemma polynomial_nat_lte_suc(n: Nat) {
    n <= n.suc
} by {
    lt_imp_lte_suc(n, n.suc)
    n <= n.suc
}

lemma polynomial_list_upper_bound_nil(s: FiniteSet[Nat]) {
    fs_from_list[Nat](List.nil[Nat]) = s implies finite_set_nat_bounded_by(s, Nat.0)
} by {
    if fs_from_list[Nat](List.nil[Nat]) = s {
        forall(k: Nat) {
            not_lt_zero(k)
            not k < Nat.0
            fs_from_list[Nat](List.nil[Nat]) = s
            s.contains(k) = fs_from_list[Nat](List.nil[Nat]).contains(k)
            fs_from_list[Nat](List.nil[Nat]).contains(k) =
                fs_from_list[Nat](List.nil[Nat]).underlying_set.contains(k)
            fs_from_list[Nat](List.nil[Nat]).underlying_set = list_set(List.nil[Nat])
            fs_from_list[Nat](List.nil[Nat]).contains(k) = list_set(List.nil[Nat]).contains(k)
            list_set_contains_eq(List.nil[Nat], k)
            list_set(List.nil[Nat]).contains(k) = List.nil[Nat].contains(k)
            List.nil[Nat].contains(k) = false
            fs_from_list[Nat](List.nil[Nat]).contains(k) = false
            not s.contains(k)
            finite_set_nat_outside_bound(s, Nat.0, k) = (not k < Nat.0 implies not s.contains(k))
            finite_set_nat_outside_bound(s, Nat.0, k)
        }
        finite_set_nat_bounded_by(s, Nat.0) = forall(k: Nat) {
            finite_set_nat_outside_bound(s, Nat.0, k)
        }
        finite_set_nat_bounded_by(s, Nat.0)
    }
}

lemma polynomial_list_bound_has_finite_set_bound(items: List[Nat], s: FiniteSet[Nat], upper: Nat) {
    fs_from_list[Nat](items) = s and is_upper_bound(items, upper) implies
    finite_set_nat_bounded_by(s, upper.suc)
} by {
    if fs_from_list[Nat](items) = s and is_upper_bound(items, upper) {
        forall(k: Nat) {
            if k < upper.suc {
                finite_set_nat_outside_bound(s, upper.suc, k) = (not k < upper.suc implies not s.contains(k))
                finite_set_nat_outside_bound(s, upper.suc, k)
            } else {
                not k < upper.suc
                if s.contains(k) {
                    fs_from_list[Nat](items) = s
                    fs_from_list[Nat](items).contains(k) = s.contains(k)
                    fs_from_list[Nat](items).contains(k)
                    fs_from_list[Nat](items).underlying_set.contains(k) = fs_from_list[Nat](items).contains(k)
                    fs_from_list[Nat](items).underlying_set.contains(k)
                    fs_from_list[Nat](items).underlying_set = list_set(items)
                    list_set(items).contains(k)
                    list_set_contains_eq(items, k)
                    list_set(items).contains(k) = items.contains(k)
                    items.contains(k)
                    upper_bound_contains(items, upper, k)
                    k <= upper
                    lt_suc(upper)
                    upper < upper.suc
                    lte_and_lt(k, upper, upper.suc)
                    k < upper.suc
                    false
                }
                not s.contains(k)
                finite_set_nat_outside_bound(s, upper.suc, k) = (not k < upper.suc implies not s.contains(k))
                finite_set_nat_outside_bound(s, upper.suc, k)
            }
        }
        finite_set_nat_bounded_by(s, upper.suc) = forall(k: Nat) {
            finite_set_nat_outside_bound(s, upper.suc, k)
        }
        finite_set_nat_bounded_by(s, upper.suc)
    }
}

/// A strict bound for all natural numbers in a list.
define list_nat_strict_bound(items: List[Nat]) -> Nat {
    match items {
        List.nil {
            Nat.0
        }
        List.cons(head, tail) {
            list_max(head, tail).suc
        }
    }
}

lemma list_nat_strict_bound_bounds_finite_set(items: List[Nat], s: FiniteSet[Nat]) {
    fs_from_list[Nat](items) = s implies finite_set_nat_bounded_by(s, list_nat_strict_bound(items))
} by {
    if fs_from_list[Nat](items) = s {
        match items {
            List.nil {
                list_nat_strict_bound(items) = Nat.0
                polynomial_list_upper_bound_nil(s)
                finite_set_nat_bounded_by(s, Nat.0)
                finite_set_nat_bounded_by(s, list_nat_strict_bound(items))
            }
            List.cons(head, tail) {
                list_nat_strict_bound(items) = list_max(head, tail).suc
                list_max_is_upper_bound(head, tail)
                is_upper_bound(List.cons(head, tail), list_max(head, tail))
                polynomial_list_bound_has_finite_set_bound(List.cons(head, tail), s, list_max(head, tail))
                finite_set_nat_bounded_by(s, list_max(head, tail).suc)
                finite_set_nat_bounded_by(s, list_nat_strict_bound(items))
            }
        }
    }
}

/// Every finite set of natural-number exponents has a strict natural bound.
theorem finite_set_nat_bounded_by_exists(s: FiniteSet[Nat]) {
    exists(n: Nat) {
        finite_set_nat_bounded_by(s, n)
    }
} by {
    finite_set_has_list[Nat](s)
    let items: List[Nat] satisfy {
        fs_from_list[Nat](items) = s
    }
    list_nat_strict_bound_bounds_finite_set(items, s)
    finite_set_nat_bounded_by(s, list_nat_strict_bound(items))
    exists(n: Nat) {
        n = list_nat_strict_bound(items) and finite_set_nat_bounded_by(s, n)
    }
}

/// Every polynomial has some strict coefficient-support bound.
theorem polynomial_support_bound_exists[R: Semiring](p: Polynomial[R]) {
    exists(n: Nat) {
        polynomial_support_bounded_by(p, n)
    }
} by {
    let s: FiniteSet[Nat] satisfy {
        lib(algebra.module.finite_support).has_support_in(p.coeff, s)
    }
    finite_set_nat_bounded_by_exists(s)
    let n: Nat satisfy {
        finite_set_nat_bounded_by(s, n)
    }
    polynomial_support_bounded_by_of_has_support_in(p, s, n)
    polynomial_support_bounded_by(p, n)
    exists(m: Nat) {
        m = n and polynomial_support_bounded_by(p, m)
    }
}

/// A sum is support-bounded when both summands are bounded by the same exponent.
theorem polynomial_add_support_bounded_by[R: Semiring](p: Polynomial[R], q: Polynomial[R], n: Nat) {
    polynomial_support_bounded_by(p, n) and polynomial_support_bounded_by(q, n) implies
    polynomial_support_bounded_by(p + q, n)
} by {
    if polynomial_support_bounded_by(p, n) and polynomial_support_bounded_by(q, n) {
        forall(k: Nat) {
            if k < n {
                coeff_zero_from_at((p + q).coeff, n, k) =
                    (not k < n implies coeff_zero_at((p + q).coeff, k))
                coeff_zero_from_at((p + q).coeff, n, k)
            } else {
                not k < n
                polynomial_support_bounded_by_apply(p, n, k)
                polynomial_support_bounded_by_apply(q, n, k)
                p.coeff(k) = R.0
                q.coeff(k) = R.0
                polynomial_add_coeff(p, q, k)
                (p + q).coeff(k) = p.coeff(k) + q.coeff(k)
                (p + q).coeff(k) = R.0 + R.0
                R.0 + R.0 = R.0
                (p + q).coeff(k) = R.0
                coeff_zero_at((p + q).coeff, k)
                polynomial_coeff_zero_from_at_intro((p + q).coeff, n, k)
                coeff_zero_from_at((p + q).coeff, n, k)
            }
        }
        polynomial_coeff_zero_from_intro((p + q).coeff, n)
        coeff_zero_from((p + q).coeff, n)
        polynomial_support_bounded_by(p + q, n)
    }
}

/// The zero polynomial has support bounded by zero.
theorem polynomial_zero_support_bounded_by_zero[R: Semiring] {
    polynomial_support_bounded_by(Polynomial[R].zero, Nat.0)
} by {
    forall(k: Nat) {
        not_lt_zero(k)
        polynomial_zero_coeff[R](k)
        coeff_zero_at(Polynomial[R].zero.coeff, k)
        polynomial_coeff_zero_from_at_intro(Polynomial[R].zero.coeff, Nat.0, k)
        coeff_zero_from_at(Polynomial[R].zero.coeff, Nat.0, k)
    }
    polynomial_coeff_zero_from_intro(Polynomial[R].zero.coeff, Nat.0)
    coeff_zero_from(Polynomial[R].zero.coeff, Nat.0)
    polynomial_support_bounded_by(Polynomial[R].zero, Nat.0)
}

/// Every bounded evaluation of the zero polynomial is zero.
theorem polynomial_eval_bound_zero[R: Semiring](x: R, n: Nat) {
    polynomial_eval_bound(Polynomial[R].zero, x, n) = R.0
} by {
    polynomial_zero_support_bounded_by_zero[R]
    polynomial_support_bounded_by(Polynomial[R].zero, Nat.0)
    polynomial_nat_lte_add_right(Nat.0, n)
    Nat.0 + n = n
    Nat.0 <= n
    polynomial_eval_bound_eq_of_support_bounded_le(Polynomial[R].zero, x, Nat.0, n)
    polynomial_eval_bound(Polynomial[R].zero, x, n) = polynomial_eval_bound(Polynomial[R].zero, x, Nat.0)
    polynomial_eval_bound(Polynomial[R].zero, x, Nat.0) = coeff_eval(Polynomial[R].zero.coeff, x, Nat.0)
    coeff_eval(Polynomial[R].zero.coeff, x, Nat.0) = R.0
    polynomial_eval_bound(Polynomial[R].zero, x, n) = R.0
}

/// The constant polynomial has support bounded by one coefficient slot.
theorem polynomial_constant_support_bounded_by_one[R: Semiring](r: R) {
    polynomial_support_bounded_by(Polynomial[R].constant(r), Nat.0.suc)
} by {
    forall(k: Nat) {
        if k < Nat.0.suc {
            coeff_zero_from_at(Polynomial[R].constant(r).coeff, Nat.0.suc, k) =
                (not k < Nat.0.suc implies coeff_zero_at(Polynomial[R].constant(r).coeff, k))
            coeff_zero_from_at(Polynomial[R].constant(r).coeff, Nat.0.suc, k)
        } else {
            not k < Nat.0.suc
            if k = Nat.0 {
                lt_suc(Nat.0)
                k < Nat.0.suc
                false
            }
            k != Nat.0
            polynomial_constant_coeff_of_ne_zero(r, k)
            Polynomial[R].constant(r).coeff(k) = R.0
            coeff_zero_at(Polynomial[R].constant(r).coeff, k)
            polynomial_coeff_zero_from_at_intro(Polynomial[R].constant(r).coeff, Nat.0.suc, k)
            coeff_zero_from_at(Polynomial[R].constant(r).coeff, Nat.0.suc, k)
        }
    }
    polynomial_coeff_zero_from_intro(Polynomial[R].constant(r).coeff, Nat.0.suc)
    coeff_zero_from(Polynomial[R].constant(r).coeff, Nat.0.suc)
    polynomial_support_bounded_by(Polynomial[R].constant(r), Nat.0.suc)
}

/// The one polynomial has support bounded by one coefficient slot.
theorem polynomial_one_support_bounded_by_one[R: Semiring] {
    polynomial_support_bounded_by(Polynomial[R].one, Nat.0.suc)
} by {
    forall(k: Nat) {
        if k < Nat.0.suc {
            coeff_zero_from_at(Polynomial[R].one.coeff, Nat.0.suc, k) =
                (not k < Nat.0.suc implies coeff_zero_at(Polynomial[R].one.coeff, k))
            coeff_zero_from_at(Polynomial[R].one.coeff, Nat.0.suc, k)
        } else {
            not k < Nat.0.suc
            if k = Nat.0 {
                lt_suc(Nat.0)
                k < Nat.0.suc
                false
            }
            k != Nat.0
            polynomial_one_coeff_of_ne_zero[R](k)
            Polynomial[R].one.coeff(k) = R.0
            coeff_zero_at(Polynomial[R].one.coeff, k)
            polynomial_coeff_zero_from_at_intro(Polynomial[R].one.coeff, Nat.0.suc, k)
            coeff_zero_from_at(Polynomial[R].one.coeff, Nat.0.suc, k)
        }
    }
    polynomial_coeff_zero_from_intro(Polynomial[R].one.coeff, Nat.0.suc)
    coeff_zero_from(Polynomial[R].one.coeff, Nat.0.suc)
    polynomial_support_bounded_by(Polynomial[R].one, Nat.0.suc)
}

/// Bounded evaluation is additive at the same bound.
theorem polynomial_eval_bound_add[R: Semiring](p: Polynomial[R], q: Polynomial[R], x: R, n: Nat) {
    polynomial_eval_bound(p + q, x, n) = polynomial_eval_bound(p, x, n) + polynomial_eval_bound(q, x, n)
} by {
    forall(k: Nat) {
        polynomial_add_coeff(p, q, k)
        pointwise_add(p.coeff, q.coeff, k) = p.coeff(k) + q.coeff(k)
        (p + q).coeff(k) = pointwise_add(p.coeff, q.coeff, k)
    }
    function_extensionality((p + q).coeff, pointwise_add(p.coeff, q.coeff))
    (p + q).coeff = pointwise_add(p.coeff, q.coeff)
    coeff_eval_add(p.coeff, q.coeff, x, n)
    coeff_eval(pointwise_add(p.coeff, q.coeff), x, n) = coeff_eval(p.coeff, x, n) + coeff_eval(q.coeff, x, n)
    polynomial_eval_bound(p + q, x, n) = coeff_eval((p + q).coeff, x, n)
    polynomial_eval_bound(p, x, n) = coeff_eval(p.coeff, x, n)
    polynomial_eval_bound(q, x, n) = coeff_eval(q.coeff, x, n)
    polynomial_eval_bound(p + q, x, n) = polynomial_eval_bound(p, x, n) + polynomial_eval_bound(q, x, n)
}

/// Evaluating a constant polynomial with one coefficient slot returns the constant.
theorem polynomial_eval_bound_constant_one[R: Semiring](r: R, x: R) {
    polynomial_eval_bound(Polynomial[R].constant(r), x, Nat.0.suc) = r
} by {
    let c = Polynomial[R].constant(r).coeff
    polynomial_eval_bound(Polynomial[R].constant(r), x, Nat.0.suc) = coeff_eval(c, x, Nat.0.suc)
    coeff_eval(c, x, Nat.0.suc) = c(Nat.0) + x * coeff_eval(coeff_tail(c), x, Nat.0)
    polynomial_constant_coeff_zero(r)
    c(Nat.0) = r
    coeff_eval(coeff_tail(c), x, Nat.0) = R.0
    x * R.0 = R.0
    c(Nat.0) + x * coeff_eval(coeff_tail(c), x, Nat.0) = r + R.0
    r + R.0 = r
    polynomial_eval_bound(Polynomial[R].constant(r), x, Nat.0.suc) = r
}

/// Evaluating the one polynomial with one coefficient slot returns one.
theorem polynomial_eval_bound_one_one[R: Semiring](x: R) {
    polynomial_eval_bound(Polynomial[R].one, x, Nat.0.suc) = R.1
} by {
    let c = Polynomial[R].one.coeff
    polynomial_eval_bound(Polynomial[R].one, x, Nat.0.suc) = coeff_eval(c, x, Nat.0.suc)
    coeff_eval(c, x, Nat.0.suc) = c(Nat.0) + x * coeff_eval(coeff_tail(c), x, Nat.0)
    polynomial_one_coeff_zero[R]
    c(Nat.0) = R.1
    coeff_eval(coeff_tail(c), x, Nat.0) = R.0
    x * R.0 = R.0
    c(Nat.0) + x * coeff_eval(coeff_tail(c), x, Nat.0) = R.1 + R.0
    R.1 + R.0 = R.1
    polynomial_eval_bound(Polynomial[R].one, x, Nat.0.suc) = R.1
}

/// Every larger bounded evaluation of a constant polynomial returns the constant.
theorem polynomial_eval_bound_constant_of_one_le[R: Semiring](r: R, x: R, n: Nat) {
    Nat.0.suc <= n implies polynomial_eval_bound(Polynomial[R].constant(r), x, n) = r
} by {
    if Nat.0.suc <= n {
        polynomial_constant_support_bounded_by_one(r)
        polynomial_eval_bound_eq_of_support_bounded_le(Polynomial[R].constant(r), x, Nat.0.suc, n)
        polynomial_eval_bound(Polynomial[R].constant(r), x, n) =
            polynomial_eval_bound(Polynomial[R].constant(r), x, Nat.0.suc)
        polynomial_eval_bound_constant_one(r, x)
        polynomial_eval_bound(Polynomial[R].constant(r), x, n) = r
    }
}

/// Every larger bounded evaluation of the one polynomial returns one.
theorem polynomial_eval_bound_one_of_one_le[R: Semiring](x: R, n: Nat) {
    Nat.0.suc <= n implies polynomial_eval_bound(Polynomial[R].one, x, n) = R.1
} by {
    if Nat.0.suc <= n {
        polynomial_one_support_bounded_by_one[R]
        polynomial_eval_bound_eq_of_support_bounded_le(Polynomial[R].one, x, Nat.0.suc, n)
        polynomial_eval_bound(Polynomial[R].one, x, n) =
            polynomial_eval_bound(Polynomial[R].one, x, Nat.0.suc)
        polynomial_eval_bound_one_one(x)
        polynomial_eval_bound(Polynomial[R].one, x, n) = R.1
    }
}

/// The bounded remainder identity for a polynomial and explicit evaluation bound.
theorem polynomial_remainder_theorem_bound[R: CommRing](p: Polynomial[R], a: R, x: R, n: Nat) {
    polynomial_eval_bound(p, x, n) =
        polynomial_eval_bound(p, a, n) +
        (x - a) * polynomial_quotient_eval_bound(p, a, x, n)
} by {
    coeff_remainder_theorem(p.coeff, a, x, n)
    coeff_eval(p.coeff, x, n) = coeff_eval(p.coeff, a, n) +
        (x - a) * coeff_eval(coeff_quotient(p.coeff, a, n), x, n)
    polynomial_eval_bound(p, x, n) = coeff_eval(p.coeff, x, n)
    polynomial_eval_bound(p, a, n) = coeff_eval(p.coeff, a, n)
    polynomial_quotient_coeff(p, a, n) = coeff_quotient(p.coeff, a, n)
    polynomial_quotient_eval_bound(p, a, x, n) =
        coeff_eval(polynomial_quotient_coeff(p, a, n), x, n)
    polynomial_quotient_eval_bound(p, a, x, n) =
        coeff_eval(coeff_quotient(p.coeff, a, n), x, n)
    polynomial_eval_bound(p, x, n) =
        polynomial_eval_bound(p, a, n) +
        (x - a) * polynomial_quotient_eval_bound(p, a, x, n)
}

/// The bounded remainder identity with the bundled synthetic quotient.
theorem polynomial_remainder_theorem_bundled_bound[R: CommRing](p: Polynomial[R], a: R, x: R, n: Nat) {
    polynomial_eval_bound(p, x, n) =
        polynomial_eval_bound(p, a, n) +
        (x - a) * polynomial_eval_bound(polynomial_quotient(p, a, n), x, n)
} by {
    polynomial_remainder_theorem_bound(p, a, x, n)
    polynomial_eval_bound(p, x, n) =
        polynomial_eval_bound(p, a, n) +
        (x - a) * polynomial_quotient_eval_bound(p, a, x, n)
    polynomial_quotient_eval_bound_eq(p, a, x, n)
    polynomial_eval_bound(polynomial_quotient(p, a, n), x, n) = polynomial_quotient_eval_bound(p, a, x, n)
    polynomial_eval_bound(p, x, n) =
        polynomial_eval_bound(p, a, n) +
        (x - a) * polynomial_eval_bound(polynomial_quotient(p, a, n), x, n)
}

/// The bundled synthetic quotient witnesses divisibility at a zero bounded value.
theorem polynomial_factor_theorem_forward_bundled_bound[R: CommRing](p: Polynomial[R], a: R, n: Nat) {
    polynomial_eval_bound(p, a, n) = R.0 implies polynomial_divisible_by_x_minus_a_polynomial_bound(p, a, n)
} by {
    if polynomial_eval_bound(p, a, n) = R.0 {
        let q = polynomial_quotient(p, a, n)
        forall(x: R) {
            polynomial_remainder_theorem_bundled_bound(p, a, x, n)
            polynomial_eval_bound(p, x, n) =
                polynomial_eval_bound(p, a, n) +
                (x - a) * polynomial_eval_bound(q, x, n)
            polynomial_eval_bound(p, a, n) = R.0
            R.0 + (x - a) * polynomial_eval_bound(q, x, n) =
                (x - a) * polynomial_eval_bound(q, x, n)
            (x - a) * polynomial_eval_bound(q, x, n) =
                polynomial_eval_bound(q, x, n) * (x - a)
            polynomial_eval_bound(p, x, n) = polynomial_eval_bound(q, x, n) * (x - a)
        }
        polynomial_x_minus_a_polynomial_witness_bound(p, a, n, q) = forall(x: R) {
            polynomial_eval_bound(p, x, n) = polynomial_eval_bound(q, x, n) * (x - a)
        }
        polynomial_x_minus_a_polynomial_witness_bound(p, a, n, q)
        exists(q0: Polynomial[R]) {
            q0 = q and polynomial_x_minus_a_polynomial_witness_bound(p, a, n, q0)
        }
        polynomial_divisible_by_x_minus_a_polynomial_bound(p, a, n)
    }
}

/// A zero bounded value gives bounded divisibility by `X - a` at the same explicit bound.
theorem polynomial_factor_theorem_forward_bound[R: CommRing](p: Polynomial[R], a: R, n: Nat) {
    polynomial_eval_bound(p, a, n) = R.0 implies polynomial_divisible_by_x_minus_a_bound(p, a, n)
} by {
    if polynomial_eval_bound(p, a, n) = R.0 {
        forall(x: R) {
            polynomial_remainder_theorem_bound(p, a, x, n)
            polynomial_eval_bound(p, x, n) =
                polynomial_eval_bound(p, a, n) +
                (x - a) * polynomial_quotient_eval_bound(p, a, x, n)
            polynomial_eval_bound(p, a, n) = R.0
            R.0 + (x - a) * polynomial_quotient_eval_bound(p, a, x, n) =
                (x - a) * polynomial_quotient_eval_bound(p, a, x, n)
            (x - a) * polynomial_quotient_eval_bound(p, a, x, n) =
                polynomial_quotient_eval_bound(p, a, x, n) * (x - a)
            polynomial_quotient_eval_bound(p, a, x, n) =
                coeff_eval(polynomial_quotient_coeff(p, a, n), x, n)
            coeff_mul_x_minus_a(polynomial_quotient_coeff(p, a, n), a, x, n) =
                coeff_eval(polynomial_quotient_coeff(p, a, n), x, n) * (x - a)
            coeff_mul_x_minus_a(polynomial_quotient_coeff(p, a, n), a, x, n) =
                polynomial_quotient_eval_bound(p, a, x, n) * (x - a)
            polynomial_eval_bound(p, x, n) =
                coeff_mul_x_minus_a(polynomial_quotient_coeff(p, a, n), a, x, n)
        }
        let q = polynomial_quotient_coeff(p, a, n)
        forall(x: R) {
            polynomial_eval_bound(p, x, n) = coeff_mul_x_minus_a(q, a, x, n)
        }
        polynomial_x_minus_a_witness_bound(p, a, n, q) = forall(x: R) {
            polynomial_eval_bound(p, x, n) = coeff_mul_x_minus_a(q, a, x, n)
        }
        polynomial_x_minus_a_witness_bound(p, a, n, q)
        exists(q0: Nat -> R) {
            q0 = q and polynomial_x_minus_a_witness_bound(p, a, n, q0)
        }
        polynomial_divisible_by_x_minus_a_bound(p, a, n)
    }
}

/// Bounded divisibility by `X - a` forces the bounded evaluation at `a` to vanish.
theorem polynomial_factor_theorem_reverse_bound[R: CommRing](p: Polynomial[R], a: R, n: Nat) {
    polynomial_divisible_by_x_minus_a_bound(p, a, n) implies polynomial_eval_bound(p, a, n) = R.0
} by {
    if polynomial_divisible_by_x_minus_a_bound(p, a, n) {
        let q: Nat -> R satisfy {
            polynomial_x_minus_a_witness_bound(p, a, n, q)
        }
        polynomial_x_minus_a_witness_bound(p, a, n, q) = forall(x: R) {
            polynomial_eval_bound(p, x, n) = coeff_mul_x_minus_a(q, a, x, n)
        }
        polynomial_eval_bound(p, a, n) = coeff_mul_x_minus_a(q, a, a, n)
        coeff_mul_x_minus_a(q, a, a, n) = coeff_eval(q, a, n) * (a - a)
        a - a = a + -a
        a + -a = R.0
        a - a = R.0
        coeff_eval(q, a, n) * R.0 = R.0
        polynomial_eval_bound(p, a, n) = R.0
    }
}

/// The bounded polynomial factor theorem for an explicit evaluation bound.
theorem polynomial_factor_theorem_bound[R: CommRing](p: Polynomial[R], a: R, n: Nat) {
    polynomial_eval_bound(p, a, n) = R.0 = polynomial_divisible_by_x_minus_a_bound(p, a, n)
} by {
    if polynomial_eval_bound(p, a, n) = R.0 {
        polynomial_factor_theorem_forward_bound(p, a, n)
        polynomial_divisible_by_x_minus_a_bound(p, a, n)
    }
    if polynomial_divisible_by_x_minus_a_bound(p, a, n) {
        polynomial_factor_theorem_reverse_bound(p, a, n)
        polynomial_eval_bound(p, a, n) = R.0
    }
}

/// Polynomial-witness bounded divisibility by `X - a` forces the bounded value at `a` to vanish.
theorem polynomial_factor_theorem_reverse_bundled_bound[R: CommRing](p: Polynomial[R], a: R, n: Nat) {
    polynomial_divisible_by_x_minus_a_polynomial_bound(p, a, n) implies polynomial_eval_bound(p, a, n) = R.0
} by {
    if polynomial_divisible_by_x_minus_a_polynomial_bound(p, a, n) {
        let q: Polynomial[R] satisfy {
            polynomial_x_minus_a_polynomial_witness_bound(p, a, n, q)
        }
        polynomial_x_minus_a_polynomial_witness_bound_apply(p, a, n, q, a)
        polynomial_eval_bound(p, a, n) = polynomial_eval_bound(q, a, n) * (a - a)
        a - a = a + -a
        a + -a = R.0
        a - a = R.0
        polynomial_eval_bound(q, a, n) * R.0 = R.0
        polynomial_eval_bound(p, a, n) = R.0
    }
}

/// The bundled bounded polynomial factor theorem for an explicit evaluation bound.
theorem polynomial_factor_theorem_bundled_bound[R: CommRing](p: Polynomial[R], a: R, n: Nat) {
    polynomial_eval_bound(p, a, n) = R.0 = polynomial_divisible_by_x_minus_a_polynomial_bound(p, a, n)
} by {
    if polynomial_eval_bound(p, a, n) = R.0 {
        polynomial_factor_theorem_forward_bundled_bound(p, a, n)
        polynomial_divisible_by_x_minus_a_polynomial_bound(p, a, n)
    }
    if polynomial_divisible_by_x_minus_a_polynomial_bound(p, a, n) {
        polynomial_factor_theorem_reverse_bundled_bound(p, a, n)
        polynomial_eval_bound(p, a, n) = R.0
    }
}

/// The bounded remainder identity may be stated at any larger bound than a support bound.
theorem polynomial_remainder_theorem_of_support_bounded_le[R: CommRing](
    p: Polynomial[R],
    a: R,
    x: R,
    n: Nat,
    m: Nat
) {
    polynomial_support_bounded_by(p, n) and n <= m implies
    polynomial_eval_bound(p, x, m) =
        polynomial_eval_bound(p, a, n) +
        (x - a) * polynomial_quotient_eval_bound(p, a, x, m)
} by {
    if polynomial_support_bounded_by(p, n) and n <= m {
        polynomial_remainder_theorem_bound(p, a, x, m)
        polynomial_eval_bound(p, x, m) =
            polynomial_eval_bound(p, a, m) +
            (x - a) * polynomial_quotient_eval_bound(p, a, x, m)
        polynomial_eval_bound_eq_of_support_bounded_le(p, a, n, m)
        polynomial_eval_bound(p, a, m) = polynomial_eval_bound(p, a, n)
        polynomial_eval_bound(p, x, m) =
            polynomial_eval_bound(p, a, n) +
            (x - a) * polynomial_quotient_eval_bound(p, a, x, m)
    }
}

/// A root at one support bound gives divisibility at every larger explicit bound.
theorem polynomial_factor_theorem_forward_of_support_bounded_le[R: CommRing](
    p: Polynomial[R],
    a: R,
    n: Nat,
    m: Nat
) {
    polynomial_support_bounded_by(p, n) and n <= m and polynomial_eval_bound(p, a, n) = R.0
    implies polynomial_divisible_by_x_minus_a_bound(p, a, m)
} by {
    if polynomial_support_bounded_by(p, n) and n <= m and polynomial_eval_bound(p, a, n) = R.0 {
        polynomial_eval_bound_eq_of_support_bounded_le(p, a, n, m)
        polynomial_eval_bound(p, a, m) = polynomial_eval_bound(p, a, n)
        polynomial_eval_bound(p, a, m) = R.0
        polynomial_factor_theorem_forward_bound(p, a, m)
        polynomial_divisible_by_x_minus_a_bound(p, a, m)
    }
}

/// Bounded divisibility at a larger explicit bound forces a root at the support bound.
theorem polynomial_factor_theorem_reverse_of_support_bounded_le[R: CommRing](
    p: Polynomial[R],
    a: R,
    n: Nat,
    m: Nat
) {
    polynomial_support_bounded_by(p, n) and n <= m and polynomial_divisible_by_x_minus_a_bound(p, a, m)
    implies polynomial_eval_bound(p, a, n) = R.0
} by {
    if polynomial_support_bounded_by(p, n) and n <= m and polynomial_divisible_by_x_minus_a_bound(p, a, m) {
        polynomial_factor_theorem_reverse_bound(p, a, m)
        polynomial_eval_bound(p, a, m) = R.0
        polynomial_eval_bound_eq_of_support_bounded_le(p, a, n, m)
        polynomial_eval_bound(p, a, m) = polynomial_eval_bound(p, a, n)
        polynomial_eval_bound(p, a, n) = R.0
    }
}

/// The bundled bounded remainder identity may be stated at any larger bound than a support bound.
theorem polynomial_remainder_theorem_bundled_of_support_bounded_le[R: CommRing](
    p: Polynomial[R],
    a: R,
    x: R,
    n: Nat,
    m: Nat
) {
    polynomial_support_bounded_by(p, n) and n <= m implies
    polynomial_eval_bound(p, x, m) =
        polynomial_eval_bound(p, a, n) +
        (x - a) * polynomial_eval_bound(polynomial_quotient(p, a, m), x, m)
} by {
    if polynomial_support_bounded_by(p, n) and n <= m {
        polynomial_remainder_theorem_bundled_bound(p, a, x, m)
        polynomial_eval_bound(p, x, m) =
            polynomial_eval_bound(p, a, m) +
            (x - a) * polynomial_eval_bound(polynomial_quotient(p, a, m), x, m)
        polynomial_eval_bound_eq_of_support_bounded_le(p, a, n, m)
        polynomial_eval_bound(p, a, m) = polynomial_eval_bound(p, a, n)
        polynomial_eval_bound(p, x, m) =
            polynomial_eval_bound(p, a, n) +
            (x - a) * polynomial_eval_bound(polynomial_quotient(p, a, m), x, m)
    }
}

/// A root at one support bound gives bundled divisibility at every larger explicit bound.
theorem polynomial_factor_theorem_forward_bundled_of_support_bounded_le[R: CommRing](
    p: Polynomial[R],
    a: R,
    n: Nat,
    m: Nat
) {
    polynomial_support_bounded_by(p, n) and n <= m and polynomial_eval_bound(p, a, n) = R.0
    implies polynomial_divisible_by_x_minus_a_polynomial_bound(p, a, m)
} by {
    if polynomial_support_bounded_by(p, n) and n <= m and polynomial_eval_bound(p, a, n) = R.0 {
        polynomial_eval_bound_eq_of_support_bounded_le(p, a, n, m)
        polynomial_eval_bound(p, a, m) = polynomial_eval_bound(p, a, n)
        polynomial_eval_bound(p, a, m) = R.0
        polynomial_factor_theorem_forward_bundled_bound(p, a, m)
        polynomial_divisible_by_x_minus_a_polynomial_bound(p, a, m)
    }
}

/// Bundled divisibility at a larger explicit bound forces a root at the support bound.
theorem polynomial_factor_theorem_reverse_bundled_of_support_bounded_le[R: CommRing](
    p: Polynomial[R],
    a: R,
    n: Nat,
    m: Nat
) {
    polynomial_support_bounded_by(p, n) and n <= m and polynomial_divisible_by_x_minus_a_polynomial_bound(p, a, m)
    implies polynomial_eval_bound(p, a, n) = R.0
} by {
    if polynomial_support_bounded_by(p, n) and n <= m and polynomial_divisible_by_x_minus_a_polynomial_bound(p, a, m) {
        polynomial_factor_theorem_reverse_bundled_bound(p, a, m)
        polynomial_eval_bound(p, a, m) = R.0
        polynomial_eval_bound_eq_of_support_bounded_le(p, a, n, m)
        polynomial_eval_bound(p, a, m) = polynomial_eval_bound(p, a, n)
        polynomial_eval_bound(p, a, n) = R.0
    }
}
