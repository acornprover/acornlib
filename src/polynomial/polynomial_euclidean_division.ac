/// Euclidean division of polynomials over a field.
///
/// For a divisor d of exact degree n (its coefficient at X^n is nonzero),
/// every polynomial p admits a quotient q and a remainder r with
/// p = q * d + r and r supported strictly below degree n.  The proof runs
/// the classical division algorithm: it inducts on a support bound m of p,
/// and when the bound m is at least n it peels off the leading monomial
/// (p(m) / d(n)) X^(m - n) of the quotient, reducing the bound of the
/// remainder by one.

from nat import Nat, alt_induction, lt_suc, lt_imp_lte_suc, lt_imp_lt_suc,
    lt_or_lte, not_lt_zero, lt_not_ref, lte_antisymm, trichotomy, add_sub
from order import lt_imp_lte
from semiring import Semiring
from comm_ring import CommRing
from algebra.ring.ring import Ring
from algebra.field.field import Field, mul_inverse_right, mul_inverse_left
from algebra.add_group import inverse_left
from polynomial import Polynomial, polynomial_constant, polynomial_monomial,
    polynomial_zero_coeff, polynomial_ext_pointwise, polynomial_mul,
    polynomial_add_assoc, polynomial_add_comm, polynomial_add_zero_left,
    polynomial_add_zero_right, polynomial_add_neg_right,
    polynomial_monomial_coeff_self,
    polynomial_support_bounded_by, polynomial_support_bounded_by_apply,
    polynomial_support_bounded_by_monotone,
    polynomial_support_bound_exists,
    polynomial_add_coeff, polynomial_sub_has_support_in_union,
    polynomial_zero_support_bounded_by_zero, polynomial_eq_zero_of_coeff_zero,
    polynomial_sub_coeff
from polynomial.polynomial_degree import polynomial_degree_le, polynomial_degree_monomial_le,
    polynomial_degree_mul_le_sum, polynomial_degree_sub_le_max, polynomial_mul_leading_coeff,
    polynomial_support_bounded_by_drop_top
from polynomial.polynomial_ring_theory import polynomial_mul_add_right
from polynomial.polynomial_divisibility import polynomial_mul_zero_left
from polynomial_mul_comm import polynomial_mul_comm

numerals Nat

/// Subtracting a polynomial and adding it back recovers the original
/// polynomial.
theorem polynomial_sub_add_cancel[R: Ring](p: Polynomial[R], t: Polynomial[R]) {
    p.sub(t) + t = p
} by {
    p.sub(t) = p + t.neg
    polynomial_add_assoc(p, t.neg, t)
    (p + t.neg) + t = p + (t.neg + t)
    polynomial_add_comm(t.neg, t)
    t.neg + t = t + t.neg
    polynomial_add_neg_right(t)
    t + t.neg = Polynomial[R].zero
    p + (t.neg + t) = p + Polynomial[R].zero
    polynomial_add_zero_right(p)
    p + Polynomial[R].zero = p
    (p + t.neg) + t = p
    p.sub(t) + t = p
}

/// The division claim for one dividend: a polynomial supported below m
/// admits a quotient and a remainder supported below n.
define division_claim_at[F: Field](d: Polynomial[F], n: Nat, m: Nat, p: Polynomial[F]) -> Bool {
    polynomial_support_bounded_by(p, m) implies
    exists(q: Polynomial[F], r: Polynomial[F]) {
        p = polynomial_mul(q, d) + r and polynomial_support_bounded_by(r, n)
    }
}

/// The division claim for every dividend supported below m.
define division_claim[F: Field](d: Polynomial[F], n: Nat, m: Nat) -> Bool {
    forall(p: Polynomial[F]) {
        division_claim_at(d, n, m, p)
    }
}

/// Applying the division claim at one dividend.
theorem division_claim_apply[F: Field](d: Polynomial[F], n: Nat, m: Nat, p: Polynomial[F]) {
    division_claim(d, n, m) implies division_claim_at(d, n, m, p)
} by {
    if division_claim(d, n, m) {
        division_claim(d, n, m) = forall(p0: Polynomial[F]) { division_claim_at(d, n, m, p0) }
        division_claim_at(d, n, m, p)
    }
}

/// Euclidean division of polynomials over a field: if d has exact degree n,
/// then p = q * d + r for some quotient q and remainder r supported strictly
/// below degree n.
theorem polynomial_euclidean_division[F: Field](p: Polynomial[F], d: Polynomial[F], n: Nat) {
    polynomial_degree_le(d, n) and d.coeff(n) != F.0
    implies exists(q: Polynomial[F], r: Polynomial[F]) {
        p = polynomial_mul(q, d) + r and polynomial_support_bounded_by(r, n)
    }
} by {
    if polynomial_degree_le(d, n) and d.coeff(n) != F.0 {
        forall(p0: Polynomial[F]) {
            if polynomial_support_bounded_by(p0, Nat.0) {
                forall(k: Nat) {
                    not_lt_zero(k)
                    polynomial_support_bounded_by_apply(p0, Nat.0, k)
                    p0.coeff(k) = F.0
                }
                polynomial_eq_zero_of_coeff_zero(p0)
                p0 = Polynomial[F].zero
                polynomial_mul_zero_left(d)
                polynomial_mul(Polynomial[F].zero, d) = Polynomial[F].zero
                polynomial_add_zero_left(Polynomial[F].zero)
                Polynomial[F].zero + Polynomial[F].zero = Polynomial[F].zero
                p0 = polynomial_mul(Polynomial[F].zero, d) + Polynomial[F].zero
                polynomial_zero_support_bounded_by_zero[F]
                polynomial_support_bounded_by(Polynomial[F].zero, Nat.0)
                polynomial_support_bounded_by_monotone(Polynomial[F].zero, Nat.0, n)
                polynomial_support_bounded_by(Polynomial[F].zero, n)
                exists(q0: Polynomial[F], r0: Polynomial[F]) {
                    q0 = Polynomial[F].zero and r0 = Polynomial[F].zero and
                    p0 = polynomial_mul(q0, d) + r0 and polynomial_support_bounded_by(r0, n)
                }
                exists(q0: Polynomial[F], r0: Polynomial[F]) {
                    p0 = polynomial_mul(q0, d) + r0 and polynomial_support_bounded_by(r0, n)
                }
                division_claim_at(d, n, Nat.0, p0)
            }
        }
        division_claim(d, n, Nat.0) = forall(p0: Polynomial[F]) { division_claim_at(d, n, Nat.0, p0) }
        division_claim(d, n, Nat.0)

        forall(m: Nat) {
            if division_claim(d, n, m) {
                forall(p0: Polynomial[F]) {
                    if polynomial_support_bounded_by(p0, m.suc) {
                        if m < n {
                            polynomial_mul_zero_left(d)
                            polynomial_mul(Polynomial[F].zero, d) = Polynomial[F].zero
                            polynomial_add_zero_left(p0)
                            Polynomial[F].zero + p0 = p0
                            p0 = polynomial_mul(Polynomial[F].zero, d) + p0
                            lt_imp_lte_suc(m, n)
                            m.suc <= n
                            polynomial_support_bounded_by_monotone(p0, m.suc, n)
                            polynomial_support_bounded_by(p0, n)
                            exists(q0: Polynomial[F], r0: Polynomial[F]) {
                                q0 = Polynomial[F].zero and r0 = p0 and
                                p0 = polynomial_mul(q0, d) + r0 and polynomial_support_bounded_by(r0, n)
                            }
                            exists(q0: Polynomial[F], r0: Polynomial[F]) {
                                p0 = polynomial_mul(q0, d) + r0 and polynomial_support_bounded_by(r0, n)
                            }
                        } else {
                            not m < n
                            lt_or_lte(m, n)
                            if m < n {
                                false
                            }
                            n <= m
                            polynomial_degree_monomial_le(m - n, p0.coeff(m) * d.coeff(n).inverse)
                            polynomial_degree_le(polynomial_monomial(m - n, p0.coeff(m) * d.coeff(n).inverse), m - n)
                            polynomial_degree_mul_le_sum(polynomial_monomial(m - n, p0.coeff(m) * d.coeff(n).inverse), d, m - n, n)
                            polynomial_degree_le(polynomial_mul(polynomial_monomial(m - n, p0.coeff(m) * d.coeff(n).inverse), d), (m - n) + n)
                            add_sub(m, n)
                            (m - n) + n = m
                            polynomial_degree_le(polynomial_mul(polynomial_monomial(m - n, p0.coeff(m) * d.coeff(n).inverse), d), m)
                            polynomial_degree_le(p0, m) = polynomial_support_bounded_by(p0, m.suc)
                            polynomial_degree_le(p0, m)
                            polynomial_degree_sub_le_max(p0, polynomial_mul(polynomial_monomial(m - n, p0.coeff(m) * d.coeff(n).inverse), d), m, m)
                            polynomial_degree_le(p0.sub(polynomial_mul(polynomial_monomial(m - n, p0.coeff(m) * d.coeff(n).inverse), d)), m.max(m))
                            m.max(m) = m
                            polynomial_degree_le(p0.sub(polynomial_mul(polynomial_monomial(m - n, p0.coeff(m) * d.coeff(n).inverse), d)), m)
                            polynomial_degree_le(p0.sub(polynomial_mul(polynomial_monomial(m - n, p0.coeff(m) * d.coeff(n).inverse), d)), m) = polynomial_support_bounded_by(p0.sub(polynomial_mul(polynomial_monomial(m - n, p0.coeff(m) * d.coeff(n).inverse), d)), m.suc)
                            polynomial_support_bounded_by(p0.sub(polynomial_mul(polynomial_monomial(m - n, p0.coeff(m) * d.coeff(n).inverse), d)), m.suc)
                            polynomial_mul_leading_coeff(polynomial_monomial(m - n, p0.coeff(m) * d.coeff(n).inverse), d, m - n, n)
                            polynomial_mul(polynomial_monomial(m - n, p0.coeff(m) * d.coeff(n).inverse), d).coeff((m - n) + n) = polynomial_monomial(m - n, p0.coeff(m) * d.coeff(n).inverse).coeff(m - n) * d.coeff(n)
                            (m - n) + n = m
                            polynomial_mul(polynomial_monomial(m - n, p0.coeff(m) * d.coeff(n).inverse), d).coeff(m) = polynomial_monomial(m - n, p0.coeff(m) * d.coeff(n).inverse).coeff(m - n) * d.coeff(n)
                            polynomial_monomial_coeff_self(m - n, p0.coeff(m) * d.coeff(n).inverse)
                            polynomial_monomial(m - n, p0.coeff(m) * d.coeff(n).inverse).coeff(m - n) = p0.coeff(m) * d.coeff(n).inverse
                            polynomial_mul(polynomial_monomial(m - n, p0.coeff(m) * d.coeff(n).inverse), d).coeff(m) = (p0.coeff(m) * d.coeff(n).inverse) * d.coeff(n)
                            (p0.coeff(m) * d.coeff(n).inverse) * d.coeff(n) = p0.coeff(m) * (d.coeff(n).inverse * d.coeff(n))
                            mul_inverse_right(d.coeff(n))
                            d.coeff(n) * d.coeff(n).inverse = F.1
                            mul_inverse_left(d.coeff(n))
                            d.coeff(n).inverse * d.coeff(n) = F.1
                            p0.coeff(m) * (d.coeff(n).inverse * d.coeff(n)) = p0.coeff(m) * F.1
                            p0.coeff(m) * F.1 = p0.coeff(m)
                            polynomial_mul(polynomial_monomial(m - n, p0.coeff(m) * d.coeff(n).inverse), d).coeff(m) = p0.coeff(m)
                            polynomial_sub_coeff(p0, polynomial_mul(polynomial_monomial(m - n, p0.coeff(m) * d.coeff(n).inverse), d), m)
                            p0.sub(polynomial_mul(polynomial_monomial(m - n, p0.coeff(m) * d.coeff(n).inverse), d)).coeff(m) = p0.coeff(m) + -polynomial_mul(polynomial_monomial(m - n, p0.coeff(m) * d.coeff(n).inverse), d).coeff(m)
                            polynomial_mul(polynomial_monomial(m - n, p0.coeff(m) * d.coeff(n).inverse), d).coeff(m) = p0.coeff(m)
                            p0.sub(polynomial_mul(polynomial_monomial(m - n, p0.coeff(m) * d.coeff(n).inverse), d)).coeff(m) = p0.coeff(m) + -p0.coeff(m)
                            inverse_left(p0.coeff(m))
                            -p0.coeff(m) + p0.coeff(m) = F.0
                            p0.coeff(m) + -p0.coeff(m) = -p0.coeff(m) + p0.coeff(m)
                            p0.coeff(m) + -p0.coeff(m) = F.0
                            p0.sub(polynomial_mul(polynomial_monomial(m - n, p0.coeff(m) * d.coeff(n).inverse), d)).coeff(m) = F.0
                            polynomial_support_bounded_by_drop_top(p0.sub(polynomial_mul(polynomial_monomial(m - n, p0.coeff(m) * d.coeff(n).inverse), d)), m)
                            polynomial_support_bounded_by(p0.sub(polynomial_mul(polynomial_monomial(m - n, p0.coeff(m) * d.coeff(n).inverse), d)), m)
                            division_claim_apply(d, n, m, p0.sub(polynomial_mul(polynomial_monomial(m - n, p0.coeff(m) * d.coeff(n).inverse), d)))
                            division_claim_at(d, n, m, p0.sub(polynomial_mul(polynomial_monomial(m - n, p0.coeff(m) * d.coeff(n).inverse), d)))
                            division_claim_at(d, n, m, p0.sub(polynomial_mul(polynomial_monomial(m - n, p0.coeff(m) * d.coeff(n).inverse), d))) = (polynomial_support_bounded_by(p0.sub(polynomial_mul(polynomial_monomial(m - n, p0.coeff(m) * d.coeff(n).inverse), d)), m) implies exists(q1: Polynomial[F], r1: Polynomial[F]) { p0.sub(polynomial_mul(polynomial_monomial(m - n, p0.coeff(m) * d.coeff(n).inverse), d)) = polynomial_mul(q1, d) + r1 and polynomial_support_bounded_by(r1, n) })
                            exists(q1: Polynomial[F], r1: Polynomial[F]) { p0.sub(polynomial_mul(polynomial_monomial(m - n, p0.coeff(m) * d.coeff(n).inverse), d)) = polynomial_mul(q1, d) + r1 and polynomial_support_bounded_by(r1, n) }
                            let q2: Polynomial[F] satisfy {
                                exists(r1: Polynomial[F]) { p0.sub(polynomial_mul(polynomial_monomial(m - n, p0.coeff(m) * d.coeff(n).inverse), d)) = polynomial_mul(q2, d) + r1 and polynomial_support_bounded_by(r1, n) }
                            }
                            let r2: Polynomial[F] satisfy {
                                p0.sub(polynomial_mul(polynomial_monomial(m - n, p0.coeff(m) * d.coeff(n).inverse), d)) = polynomial_mul(q2, d) + r2 and polynomial_support_bounded_by(r2, n)
                            }
                            p0.sub(polynomial_mul(polynomial_monomial(m - n, p0.coeff(m) * d.coeff(n).inverse), d)) = polynomial_mul(q2, d) + r2
                            polynomial_sub_add_cancel(p0, polynomial_mul(polynomial_monomial(m - n, p0.coeff(m) * d.coeff(n).inverse), d))
                            p0.sub(polynomial_mul(polynomial_monomial(m - n, p0.coeff(m) * d.coeff(n).inverse), d)) + polynomial_mul(polynomial_monomial(m - n, p0.coeff(m) * d.coeff(n).inverse), d) = p0
                            polynomial_add_comm(polynomial_mul(polynomial_monomial(m - n, p0.coeff(m) * d.coeff(n).inverse), d), p0.sub(polynomial_mul(polynomial_monomial(m - n, p0.coeff(m) * d.coeff(n).inverse), d)))
                            polynomial_mul(polynomial_monomial(m - n, p0.coeff(m) * d.coeff(n).inverse), d) + p0.sub(polynomial_mul(polynomial_monomial(m - n, p0.coeff(m) * d.coeff(n).inverse), d)) = p0
                            polynomial_mul(polynomial_monomial(m - n, p0.coeff(m) * d.coeff(n).inverse), d) + (polynomial_mul(q2, d) + r2) = p0
                            polynomial_add_assoc(polynomial_mul(polynomial_monomial(m - n, p0.coeff(m) * d.coeff(n).inverse), d), polynomial_mul(q2, d), r2)
                            (polynomial_mul(polynomial_monomial(m - n, p0.coeff(m) * d.coeff(n).inverse), d) + polynomial_mul(q2, d)) + r2 = polynomial_mul(polynomial_monomial(m - n, p0.coeff(m) * d.coeff(n).inverse), d) + (polynomial_mul(q2, d) + r2)
                            polynomial_mul_add_right(polynomial_monomial(m - n, p0.coeff(m) * d.coeff(n).inverse), q2, d)
                            polynomial_mul(polynomial_monomial(m - n, p0.coeff(m) * d.coeff(n).inverse) + q2, d) = polynomial_mul(polynomial_monomial(m - n, p0.coeff(m) * d.coeff(n).inverse), d) + polynomial_mul(q2, d)
                            polynomial_mul(polynomial_monomial(m - n, p0.coeff(m) * d.coeff(n).inverse) + q2, d) + r2 = p0
                            p0 = polynomial_mul(polynomial_monomial(m - n, p0.coeff(m) * d.coeff(n).inverse) + q2, d) + r2
                            polynomial_support_bounded_by(r2, n)
                            exists(q0: Polynomial[F], r0: Polynomial[F]) {
                                q0 = polynomial_monomial(m - n, p0.coeff(m) * d.coeff(n).inverse) + q2 and r0 = r2 and
                                p0 = polynomial_mul(q0, d) + r0 and polynomial_support_bounded_by(r0, n)
                            }
                            exists(q0: Polynomial[F], r0: Polynomial[F]) {
                                p0 = polynomial_mul(q0, d) + r0 and polynomial_support_bounded_by(r0, n)
                            }
                        }
                        division_claim_at(d, n, m.suc, p0)
                    }
                }
                forall(p0: Polynomial[F]) {
                    division_claim_at(d, n, m.suc, p0) = (polynomial_support_bounded_by(p0, m.suc) implies exists(q0: Polynomial[F], r0: Polynomial[F]) { p0 = polynomial_mul(q0, d) + r0 and polynomial_support_bounded_by(r0, n) })
                    division_claim_at(d, n, m.suc, p0)
                }
                division_claim(d, n, m.suc) = forall(p0: Polynomial[F]) { division_claim_at(d, n, m.suc, p0) }
                division_claim(d, n, m.suc)
            }
        }

        division_claim(d, n, Nat.0) and forall(m: Nat) { division_claim(d, n, m) implies division_claim(d, n, m.suc) }
        alt_induction(division_claim(d, n))
        forall(m: Nat) { division_claim(d, n, m) }
        polynomial_support_bound_exists(p)
        let m: Nat satisfy {
            polynomial_support_bounded_by(p, m)
        }
        division_claim(d, n, m)
        division_claim_apply(d, n, m, p)
        division_claim_at(d, n, m, p)
        division_claim_at(d, n, m, p) = (polynomial_support_bounded_by(p, m) implies exists(q: Polynomial[F], r: Polynomial[F]) { p = polynomial_mul(q, d) + r and polynomial_support_bounded_by(r, n) })
        exists(q: Polynomial[F], r: Polynomial[F]) {
            p = polynomial_mul(q, d) + r and polynomial_support_bounded_by(r, n)
        }
    }
}
