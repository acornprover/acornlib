/// Deepening multiplicity theory: the coefficient-level factor theorem
/// (a root of `p` is exactly a point where `X - a` divides `p`), root
/// multiplicity as divisibility by powers of the linear factor, the
/// divisibility of `p` by the product of the distinct linear factors of its
/// roots, and the classical root-count bound in degree form.
///
/// The coefficient-level factor theorem is proved from the synthetic
/// quotient: at a support bound `n` the bundled quotient
/// `polynomial_quotient(p, a, n.suc)` recovers `p` as `(X - a) * q`, so a
/// root gives an actual polynomial witness for divisibility by `X - a`.

from nat import Nat, alt_induction, lt_suc, lt_imp_lte_suc, lt_imp_lt_suc, lte_suc_suc,
    lt_cancel_suc, lt_trans, lt_or_lte, lte_and_lt, lte_imp_not_lt, lt_not_symm,
    lte_trans, lte_antisymm, zero_or_suc, not_lt_zero, alt_suc_ne_zero, suc_sub_one,
    sub_zero, sub_self, add_sub
from order import lt_imp_lte
from semiring import Semiring
from comm_ring import CommRing
from algebra.field.field import Field, field_mul_eq_zero, field_mul_nonzero
from algebra.ring.ring import mul_neg_left
from list import List, unique_implies_tail_unique, unique_length
from polynomial import Polynomial, polynomial_constant, polynomial_monomial, polynomial_eval,
    polynomial_eval_mul, polynomial_mul, polynomial_mul_coeff, polynomial_mul_coeff_apply,
    polynomial_mul_term_coeff,
    polynomial_zero_coeff, polynomial_support_bound_exists, polynomial_add_coeff,
    polynomial_constant_coeff_of_ne_zero, polynomial_monomial_coeff_of_ne,
    polynomial_roots_on_list, polynomial_root_list_length_lt_support_bound,
    polynomial_ext_pointwise, polynomial_quotient, polynomial_quotient_coeff,
    polynomial_quotient_coeff_at,
    polynomial_quotient_support_bounded_by, polynomial_quotient_support_bounded_by_pred,
    polynomial_support_bounded_by, polynomial_support_bounded_by_apply,
    polynomial_support_bounded_by_monotone, polynomial_eval_bound,
    polynomial_eval_bound_eq_coeff_eval, polynomial_eval_eq_eval_bound_of_support_bounded,
    coeff_eval, coeff_tail, coeff_quotient, coeff_quotient_zero_index, polynomial_eval_constant
from polynomial.polynomial_vieta import linear_polynomial, linear_polynomial_eval,
    linear_polynomial_neg_root, linear_polynomial_coeff_zero, linear_polynomial_coeff_one
from polynomial.polynomial_roots import polynomial_sub_eq_zero_iff
from polynomial.polynomial_divisibility import polynomial_divides, polynomial_divides_eval_witness,
    polynomial_divides_refl
from polynomial.polynomial_degree import polynomial_degree_le
from polynomial_mul_comm import mul_coeff_eq_range_sum, polynomial_mul_one_left,
    polynomial_mul_one_right, polynomial_mul_comm
from polynomial_mul_assoc import polynomial_mul_assoc
from data.nat.nat_range_sum import range_sum, range_sum_suc, range_sum_one

numerals Nat

// ---------------------------------------------------------------------------
// Polynomial powers
// ---------------------------------------------------------------------------

/// The `k`-th power of a polynomial, `p^k`.
define polynomial_pow[R: Semiring](p: Polynomial[R], k: Nat) -> Polynomial[R] {
    match k {
        Nat.zero {
            Polynomial[R].one
        }
        Nat.suc(pred) {
            polynomial_mul(p, polynomial_pow(p, pred))
        }
    }
}

/// The zeroth power of a polynomial is one.
theorem polynomial_pow_zero[R: Semiring](p: Polynomial[R]) {
    polynomial_pow(p, Nat.0) = Polynomial[R].one
} by {
    polynomial_pow(p, Nat.0) = Polynomial[R].one
}

/// A successor power is the base times the previous power.
theorem polynomial_pow_suc[R: Semiring](p: Polynomial[R], k: Nat) {
    polynomial_pow(p, k.suc) = polynomial_mul(p, polynomial_pow(p, k))
} by {
    polynomial_pow(p, k.suc) = polynomial_mul(p, polynomial_pow(p, k))
}

/// The first power of a polynomial is itself.
theorem polynomial_pow_one[R: CommRing](p: Polynomial[R]) {
    polynomial_pow(p, Nat.1) = p
} by {
    polynomial_pow(p, Nat.1) = polynomial_pow(p, Nat.0.suc)
    polynomial_pow_suc(p, Nat.0)
    polynomial_pow(p, Nat.0.suc) = polynomial_mul(p, polynomial_pow(p, Nat.0))
    polynomial_pow_zero(p)
    polynomial_pow(p, Nat.0) = Polynomial[R].one
    polynomial_mul(p, polynomial_pow(p, Nat.0)) = polynomial_mul(p, Polynomial[R].one)
    polynomial_mul_one_right(p)
    polynomial_mul(p, Polynomial[R].one) = p
    polynomial_pow(p, Nat.1) = p
}

/// Powers add: `p^(m + n) = p^m * p^n`.
theorem polynomial_pow_add[R: CommRing](p: Polynomial[R], m: Nat, n: Nat) {
    polynomial_pow(p, m + n) = polynomial_mul(polynomial_pow(p, m), polynomial_pow(p, n))
} by {
    define statement(k: Nat) -> Bool {
        polynomial_pow(p, k + n) = polynomial_mul(polynomial_pow(p, k), polynomial_pow(p, n))
    }

    polynomial_pow(p, Nat.0 + n) = polynomial_pow(p, n)
    polynomial_pow_zero(p)
    polynomial_pow(p, Nat.0) = Polynomial[R].one
    polynomial_mul_one_left(polynomial_pow(p, n))
    polynomial_mul(Polynomial[R].one, polynomial_pow(p, n)) = polynomial_pow(p, n)
    polynomial_mul(polynomial_pow(p, Nat.0), polynomial_pow(p, n)) = polynomial_pow(p, n)
    statement(Nat.0)

    forall(k: Nat) {
        if statement(k) {
            polynomial_pow(p, k.suc + n) = polynomial_pow(p, (k + n).suc)
            polynomial_pow_suc(p, k + n)
            polynomial_pow(p, (k + n).suc) = polynomial_mul(p, polynomial_pow(p, k + n))
            statement(k) = (polynomial_pow(p, k + n) =
                polynomial_mul(polynomial_pow(p, k), polynomial_pow(p, n)))
            polynomial_pow(p, k + n) = polynomial_mul(polynomial_pow(p, k), polynomial_pow(p, n))
            polynomial_mul(p, polynomial_pow(p, k + n)) =
                polynomial_mul(p, polynomial_mul(polynomial_pow(p, k), polynomial_pow(p, n)))
            polynomial_mul_assoc(p, polynomial_pow(p, k), polynomial_pow(p, n))
            polynomial_mul(polynomial_mul(p, polynomial_pow(p, k)), polynomial_pow(p, n)) =
                polynomial_mul(p, polynomial_mul(polynomial_pow(p, k), polynomial_pow(p, n)))
            polynomial_pow_suc(p, k)
            polynomial_pow(p, k.suc) = polynomial_mul(p, polynomial_pow(p, k))
            polynomial_mul(polynomial_pow(p, k.suc), polynomial_pow(p, n)) =
                polynomial_mul(polynomial_mul(p, polynomial_pow(p, k)), polynomial_pow(p, n))
            polynomial_pow(p, k.suc + n) =
                polynomial_mul(polynomial_pow(p, k.suc), polynomial_pow(p, n))
            statement(k.suc)
        }
    }

    statement(Nat.0) and forall(k: Nat) {
        statement(k) implies statement(k.suc)
    }
    alt_induction(statement)
    forall(k: Nat) { statement(k) }
    statement(m)
}

// ---------------------------------------------------------------------------
// Range sums of functions that vanish away from the first two points
// ---------------------------------------------------------------------------

/// A range sum of a function that vanishes from the third index on is the sum
/// of its first two values.
lemma multiplicity_range_sum_first_two[R: Semiring](f: Nat -> R, n: Nat) {
    Nat.2 <= n and (forall(i: Nat) { Nat.2 <= i and i < n implies f(i) = R.0 }) implies
    range_sum(f, n) = f(Nat.0) + f(Nat.1)
} by {
    define statement(k: Nat) -> Bool {
        Nat.2 <= k and (forall(i: Nat) { Nat.2 <= i and i < k implies f(i) = R.0 }) implies
        range_sum(f, k) = f(Nat.0) + f(Nat.1)
    }

    if Nat.2 <= Nat.0 {
        lt_suc(Nat.0)
        Nat.0 < Nat.0.suc
        Nat.0.suc = Nat.1
        lt_suc(Nat.1)
        Nat.1 < Nat.1.suc
        Nat.1.suc = Nat.2
        lt_trans(Nat.0, Nat.1, Nat.2)
        Nat.0 < Nat.2
        lte_imp_not_lt(Nat.2, Nat.0)
        not Nat.0 < Nat.2
        false
    }
    statement(Nat.0)

    forall(k: Nat) {
        if statement(k) {
            statement(k.suc) = (Nat.2 <= k.suc and
                (forall(i: Nat) { Nat.2 <= i and i < k.suc implies f(i) = R.0 }) implies
                range_sum(f, k.suc) = f(Nat.0) + f(Nat.1))
            if Nat.2 <= k.suc and forall(i: Nat) { Nat.2 <= i and i < k.suc implies f(i) = R.0 } {
                lt_suc(k)
                k < k.suc
                if Nat.2 <= k {
                    statement(k)
                    statement(k) = (Nat.2 <= k and
                        (forall(i: Nat) { Nat.2 <= i and i < k implies f(i) = R.0 }) implies
                        range_sum(f, k) = f(Nat.0) + f(Nat.1))
                    forall(i: Nat) {
                        if Nat.2 <= i and i < k {
                            lt_suc(k)
                            k < k.suc
                            lte_and_lt(Nat.2, i, k.suc)
                            Nat.2 <= i and i < k.suc
                            f(i) = R.0
                        }
                    }
                    range_sum(f, k) = f(Nat.0) + f(Nat.1)
                    range_sum_suc(f, k)
                    range_sum(f, k.suc) = range_sum(f, k) + f(k)
                    range_sum(f, k.suc) = f(Nat.0) + f(Nat.1) + f(k)
                    Nat.2 <= k
                    lt_suc(k)
                    k < k.suc
                    f(k) = R.0
                    range_sum(f, k.suc) = f(Nat.0) + f(Nat.1) + R.0
                    f(Nat.0) + f(Nat.1) + R.0 = f(Nat.0) + f(Nat.1)
                    range_sum(f, k.suc) = f(Nat.0) + f(Nat.1)
                } else {
                    not Nat.2 <= k
                    if k.suc = Nat.2 {
                        range_sum_suc(f, Nat.1)
                        range_sum(f, Nat.2) = range_sum(f, Nat.1) + f(Nat.1)
                        range_sum_one(f)
                        range_sum(f, Nat.1) = f(Nat.0)
                        range_sum(f, Nat.2) = f(Nat.0) + f(Nat.1)
                        k.suc = Nat.2
                        range_sum(f, k.suc) = f(Nat.0) + f(Nat.1)
                    } else {
                        k.suc != Nat.2
                        lt_or_lte(k, Nat.2)
                        if k < Nat.2 {
                            lt_imp_lte_suc(k, Nat.2)
                            k.suc <= Nat.2
                            lte_antisymm(k.suc, Nat.2)
                            k.suc = Nat.2
                            false
                        }
                        if Nat.2 <= k {
                            false
                        }
                        false
                    }
                }
                statement(k.suc)
            }
            statement(k.suc)
        }
    }

    statement(Nat.0) and forall(k: Nat) {
        statement(k) implies statement(k.suc)
    }
    alt_induction(statement)
    forall(k: Nat) { statement(k) }
    statement(n)
}

/// A linear polynomial has zero coefficient from the third slot on.
lemma multiplicity_linear_coeff_zero_ge_two[R: Semiring](a: R, b: R, i: Nat) {
    Nat.2 <= i implies linear_polynomial(a, b).coeff(i) = R.0
} by {
    if Nat.2 <= i {
        if i = Nat.0 {
            Nat.2 <= i
            i = Nat.0
            Nat.2 <= Nat.0
            lte_imp_not_lt(Nat.2, Nat.0)
            not Nat.0 < Nat.2
            lt_suc(Nat.0)
            Nat.0 < Nat.0.suc
            Nat.0.suc = Nat.1
            lt_suc(Nat.1)
            Nat.1 < Nat.1.suc
            Nat.1.suc = Nat.2
            lt_trans(Nat.0, Nat.1, Nat.2)
            Nat.0 < Nat.2
            false
        }
        i != Nat.0
        if i = Nat.1 {
            Nat.2 <= i
            i = Nat.1
            Nat.2 <= Nat.1
            lte_imp_not_lt(Nat.2, Nat.1)
            not Nat.1 < Nat.2
            lt_suc(Nat.1)
            Nat.1 < Nat.1.suc
            Nat.1.suc = Nat.2
            Nat.1 < Nat.2
            false
        }
        i != Nat.1
        polynomial_add_coeff(polynomial_constant(b), polynomial_monomial(Nat.1, a), i)
        linear_polynomial(a, b).coeff(i) =
            polynomial_constant(b).coeff(i) + polynomial_monomial(Nat.1, a).coeff(i)
        polynomial_constant_coeff_of_ne_zero(b, i)
        polynomial_constant(b).coeff(i) = R.0
        polynomial_monomial_coeff_of_ne[R](Nat.1, a, i)
        polynomial_monomial(Nat.1, a).coeff(i) = R.0
        linear_polynomial(a, b).coeff(i) = R.0 + R.0
        R.0 + R.0 = R.0
        linear_polynomial(a, b).coeff(i) = R.0
    }
}

/// The coefficient of `(a * X + b) * q` at a positive index is the sum of the
/// two contributing convolution terms.
theorem polynomial_mul_linear_coeff[R: CommRing](a: R, b: R, q: Polynomial[R], k: Nat) {
    k >= Nat.1 implies
    polynomial_mul(linear_polynomial(a, b), q).coeff(k) =
        linear_polynomial(a, b).coeff(Nat.0) * q.coeff(k) +
        linear_polynomial(a, b).coeff(Nat.1) * q.coeff(k - Nat.1)
} by {
    if k >= Nat.1 {
        polynomial_mul_coeff_apply(linear_polynomial(a, b), q, k)
        polynomial_mul(linear_polynomial(a, b), q).coeff(k) =
            polynomial_mul_coeff(linear_polynomial(a, b), q, k)
        mul_coeff_eq_range_sum(linear_polynomial(a, b), q, k)
        polynomial_mul_coeff(linear_polynomial(a, b), q, k) =
            range_sum(polynomial_mul_term_coeff(linear_polynomial(a, b), q, k), k.suc)
        forall(i: Nat) {
            if Nat.2 <= i and i < k.suc {
                multiplicity_linear_coeff_zero_ge_two(a, b, i)
                linear_polynomial(a, b).coeff(i) = R.0
                polynomial_mul_term_coeff(linear_polynomial(a, b), q, k, i) =
                    linear_polynomial(a, b).coeff(i) * q.coeff(k - i)
                polynomial_mul_term_coeff(linear_polynomial(a, b), q, k, i) = R.0 * q.coeff(k - i)
                R.0 * q.coeff(k - i) = R.0
                polynomial_mul_term_coeff(linear_polynomial(a, b), q, k, i) = R.0
            }
        }
        lt_imp_lte_suc(Nat.1, k)
        Nat.2 <= k.suc
        multiplicity_range_sum_first_two(polynomial_mul_term_coeff(linear_polynomial(a, b), q, k), k.suc)
        range_sum(polynomial_mul_term_coeff(linear_polynomial(a, b), q, k), k.suc) =
            polynomial_mul_term_coeff(linear_polynomial(a, b), q, k, Nat.0) +
            polynomial_mul_term_coeff(linear_polynomial(a, b), q, k, Nat.1)
        polynomial_mul_coeff(linear_polynomial(a, b), q, k) =
            polynomial_mul_term_coeff(linear_polynomial(a, b), q, k, Nat.0) +
            polynomial_mul_term_coeff(linear_polynomial(a, b), q, k, Nat.1)
        polynomial_mul_term_coeff(linear_polynomial(a, b), q, k, Nat.0) =
            linear_polynomial(a, b).coeff(Nat.0) * q.coeff(k - Nat.0)
        sub_zero(k)
        k - Nat.0 = k
        polynomial_mul_term_coeff(linear_polynomial(a, b), q, k, Nat.0) =
            linear_polynomial(a, b).coeff(Nat.0) * q.coeff(k)
        polynomial_mul_term_coeff(linear_polynomial(a, b), q, k, Nat.1) =
            linear_polynomial(a, b).coeff(Nat.1) * q.coeff(k - Nat.1)
        polynomial_mul_coeff(linear_polynomial(a, b), q, k) =
            linear_polynomial(a, b).coeff(Nat.0) * q.coeff(k) +
            linear_polynomial(a, b).coeff(Nat.1) * q.coeff(k - Nat.1)
        polynomial_mul(linear_polynomial(a, b), q).coeff(k) =
            linear_polynomial(a, b).coeff(Nat.0) * q.coeff(k) +
            linear_polynomial(a, b).coeff(Nat.1) * q.coeff(k - Nat.1)
    }
}

/// The coefficient of `(X - a) * q` at a positive index.
theorem polynomial_mul_x_minus_a_coeff[R: CommRing](a: R, q: Polynomial[R], k: Nat) {
    k >= Nat.1 implies
    polynomial_mul(linear_polynomial(R.1, -a), q).coeff(k) =
        (-a) * q.coeff(k) + q.coeff(k - Nat.1)
} by {
    if k >= Nat.1 {
        polynomial_mul_linear_coeff(R.1, -a, q, k)
        polynomial_mul(linear_polynomial(R.1, -a), q).coeff(k) =
            linear_polynomial(R.1, -a).coeff(Nat.0) * q.coeff(k) +
            linear_polynomial(R.1, -a).coeff(Nat.1) * q.coeff(k - Nat.1)
        linear_polynomial_coeff_zero(R.1, -a)
        linear_polynomial(R.1, -a).coeff(Nat.0) = -a
        linear_polynomial_coeff_one(R.1, -a)
        linear_polynomial(R.1, -a).coeff(Nat.1) = R.1
        polynomial_mul(linear_polynomial(R.1, -a), q).coeff(k) = (-a) * q.coeff(k) + R.1 * q.coeff(k - Nat.1)
        R.1 * q.coeff(k - Nat.1) = q.coeff(k - Nat.1)
        polynomial_mul(linear_polynomial(R.1, -a), q).coeff(k) = (-a) * q.coeff(k) + q.coeff(k - Nat.1)
    }
}

/// The constant coefficient of `(X - a) * q`.
theorem polynomial_mul_x_minus_a_coeff_zero[R: CommRing](a: R, q: Polynomial[R]) {
    polynomial_mul(linear_polynomial(R.1, -a), q).coeff(Nat.0) = (-a) * q.coeff(Nat.0)
} by {
    polynomial_mul_coeff_apply(linear_polynomial(R.1, -a), q, Nat.0)
    polynomial_mul(linear_polynomial(R.1, -a), q).coeff(Nat.0) =
        polynomial_mul_coeff(linear_polynomial(R.1, -a), q, Nat.0)
    mul_coeff_eq_range_sum(linear_polynomial(R.1, -a), q, Nat.0)
    polynomial_mul_coeff(linear_polynomial(R.1, -a), q, Nat.0) =
        range_sum(polynomial_mul_term_coeff(linear_polynomial(R.1, -a), q, Nat.0), Nat.1)
    range_sum_one(polynomial_mul_term_coeff(linear_polynomial(R.1, -a), q, Nat.0))
    range_sum(polynomial_mul_term_coeff(linear_polynomial(R.1, -a), q, Nat.0), Nat.1) =
        polynomial_mul_term_coeff(linear_polynomial(R.1, -a), q, Nat.0, Nat.0)
    polynomial_mul_coeff(linear_polynomial(R.1, -a), q, Nat.0) =
        polynomial_mul_term_coeff(linear_polynomial(R.1, -a), q, Nat.0, Nat.0)
    polynomial_mul_term_coeff(linear_polynomial(R.1, -a), q, Nat.0, Nat.0) =
        linear_polynomial(R.1, -a).coeff(Nat.0) * q.coeff(Nat.0 - Nat.0)
    sub_self(Nat.0)
    Nat.0 - Nat.0 = Nat.0
    linear_polynomial_coeff_zero(R.1, -a)
    linear_polynomial(R.1, -a).coeff(Nat.0) = -a
    polynomial_mul_term_coeff(linear_polynomial(R.1, -a), q, Nat.0, Nat.0) = (-a) * q.coeff(Nat.0)
    polynomial_mul(linear_polynomial(R.1, -a), q).coeff(Nat.0) = (-a) * q.coeff(Nat.0)
}

// ---------------------------------------------------------------------------
// The synthetic quotient recurrence and the coefficient-level factor theorem
// ---------------------------------------------------------------------------

/// The synthetic quotient satisfies the backwards recurrence that encodes
/// division by `X - a`: `q_i = p_{i + 1} + a * q_{i + 1}`.
theorem multiplicity_coeff_quotient_recurrence[R: Semiring](c: Nat -> R, a: R, n: Nat, i: Nat) {
    i < n implies
    coeff_quotient(c, a, n.suc, i) = c(i.suc) + a * coeff_quotient(c, a, n.suc, i.suc)
} by {
    define statement(m: Nat) -> Bool {
        forall(d: Nat -> R, b: R, j: Nat) {
            j < m implies
            coeff_quotient(d, b, m.suc, j) = d(j.suc) + b * coeff_quotient(d, b, m.suc, j.suc)
        }
    }

    forall(d: Nat -> R, b: R, j: Nat) {
        if j < Nat.0 {
            not_lt_zero(j)
            false
        }
    }
    statement(Nat.0)

    forall(k: Nat) {
        if statement(k) {
            forall(d: Nat -> R, b: R, j: Nat) {
                if j < k.suc {
                    if j = Nat.0 {
                        coeff_quotient(d, b, k.suc.suc, Nat.0) =
                            coeff_eval(coeff_tail(d), b, k.suc)
                        coeff_eval(coeff_tail(d), b, k.suc) =
                            coeff_tail(d, Nat.0) + b * coeff_eval(coeff_tail(coeff_tail(d)), b, k)
                        coeff_tail(d, Nat.0) = d(Nat.0.suc)
                        coeff_quotient(d, b, k.suc.suc, Nat.1) =
                            coeff_quotient(coeff_tail(d), b, k.suc, Nat.0)
                        coeff_quotient_zero_index(coeff_tail(d), b, k)
                        coeff_quotient(coeff_tail(d), b, k.suc, Nat.0) =
                            coeff_eval(coeff_tail(coeff_tail(d)), b, k)
                        b * coeff_quotient(d, b, k.suc.suc, Nat.1) =
                            b * coeff_eval(coeff_tail(coeff_tail(d)), b, k)
                        coeff_quotient(d, b, k.suc.suc, Nat.0) =
                            d(Nat.0.suc) + b * coeff_quotient(d, b, k.suc.suc, Nat.1)
                        coeff_quotient(d, b, k.suc.suc, j) =
                            d(j.suc) + b * coeff_quotient(d, b, k.suc.suc, j.suc)
                    } else {
                        j != Nat.0
                        zero_or_suc(j)
                        let jp: Nat satisfy {
                            j = jp.suc
                        }
                        j = jp.suc
                        jp.suc < k.suc
                        lt_cancel_suc(jp, k)
                        jp < k
                        statement(k) = forall(e: Nat -> R, x: R, m: Nat) {
                            m < k implies
                            coeff_quotient(e, x, k.suc, m) =
                                e(m.suc) + x * coeff_quotient(e, x, k.suc, m.suc)
                        }
                        coeff_quotient(coeff_tail(d), b, k.suc, jp) =
                            coeff_tail(d, jp.suc) + b * coeff_quotient(coeff_tail(d), b, k.suc, jp.suc)
                        coeff_quotient(d, b, k.suc.suc, jp.suc) =
                            coeff_quotient(coeff_tail(d), b, k.suc, jp)
                        coeff_quotient(d, b, k.suc.suc, jp.suc.suc) =
                            coeff_quotient(coeff_tail(d), b, k.suc, jp.suc)
                        coeff_tail(d, jp.suc) = d(jp.suc.suc)
                        coeff_quotient(d, b, k.suc.suc, j) =
                            d(j.suc) + b * coeff_quotient(d, b, k.suc.suc, j.suc)
                    }
                }
            }
            statement(k.suc)
        }
    }

    statement(Nat.0) and forall(k: Nat) {
        statement(k) implies statement(k.suc)
    }
    alt_induction(statement)
    forall(k: Nat) { statement(k) }
    statement(n)
    statement(n) = forall(d: Nat -> R, b: R, j: Nat) {
        j < n implies
        coeff_quotient(d, b, n.suc, j) = d(j.suc) + b * coeff_quotient(d, b, n.suc, j.suc)
    }
    coeff_quotient(c, a, n.suc, i) = c(i.suc) + a * coeff_quotient(c, a, n.suc, i.suc)
}

/// The bundled synthetic quotient satisfies the division recurrence.
theorem multiplicity_quotient_recurrence[R: Semiring](p: Polynomial[R], a: R, n: Nat, i: Nat) {
    i < n implies
    polynomial_quotient(p, a, n.suc).coeff(i) =
        p.coeff(i.suc) + a * polynomial_quotient(p, a, n.suc).coeff(i.suc)
} by {
    if i < n {
        polynomial_quotient_coeff_at(p, a, n.suc, i)
        polynomial_quotient(p, a, n.suc).coeff(i) = polynomial_quotient_coeff(p, a, n.suc, i)
        polynomial_quotient_coeff(p, a, n.suc, i) = coeff_quotient(p.coeff, a, n.suc, i)
        multiplicity_coeff_quotient_recurrence(p.coeff, a, n, i)
        coeff_quotient(p.coeff, a, n.suc, i) = p.coeff(i.suc) + a * coeff_quotient(p.coeff, a, n.suc, i.suc)
        polynomial_quotient_coeff_at(p, a, n.suc, i.suc)
        polynomial_quotient(p, a, n.suc).coeff(i.suc) = polynomial_quotient_coeff(p, a, n.suc, i.suc)
        polynomial_quotient_coeff(p, a, n.suc, i.suc) = coeff_quotient(p.coeff, a, n.suc, i.suc)
        polynomial_quotient(p, a, n.suc).coeff(i) =
            p.coeff(i.suc) + a * polynomial_quotient(p, a, n.suc).coeff(i.suc)
    }
}

/// The constant coefficient of the bundled synthetic quotient is the tail
/// evaluated at the root.
theorem multiplicity_quotient_zero_coeff[R: Semiring](p: Polynomial[R], a: R, n: Nat) {
    polynomial_quotient(p, a, n.suc).coeff(Nat.0) = coeff_eval(coeff_tail(p.coeff), a, n)
} by {
    polynomial_quotient_coeff_at(p, a, n.suc, Nat.0)
    polynomial_quotient(p, a, n.suc).coeff(Nat.0) = polynomial_quotient_coeff(p, a, n.suc, Nat.0)
    polynomial_quotient_coeff(p, a, n.suc, Nat.0) = coeff_quotient(p.coeff, a, n.suc, Nat.0)
    coeff_quotient_zero_index(p.coeff, a, n)
    coeff_quotient(p.coeff, a, n.suc, Nat.0) = coeff_eval(coeff_tail(p.coeff), a, n)
    polynomial_quotient(p, a, n.suc).coeff(Nat.0) = coeff_eval(coeff_tail(p.coeff), a, n)
}

/// The bundled synthetic quotient vanishes at its diagonal index.
theorem multiplicity_quotient_diagonal_zero[R: Semiring](p: Polynomial[R], a: R, n: Nat) {
    polynomial_quotient(p, a, n.suc).coeff(n) = R.0
} by {
    polynomial_quotient_coeff_at(p, a, n.suc, n)
    polynomial_quotient(p, a, n.suc).coeff(n) = polynomial_quotient_coeff(p, a, n.suc, n)
    polynomial_quotient_coeff(p, a, n.suc, n) = coeff_quotient(p.coeff, a, n.suc, n)
    define statement(m: Nat) -> Bool {
        forall(d: Nat -> R, b: R) {
            coeff_quotient(d, b, m.suc, m) = R.0
        }
    }
    forall(d: Nat -> R, b: R) {
        coeff_quotient(d, b, Nat.0.suc, Nat.0) = coeff_eval(coeff_tail(d), b, Nat.0)
        coeff_eval(coeff_tail(d), b, Nat.0) = R.0
        coeff_quotient(d, b, Nat.0.suc, Nat.0) = R.0
    }
    statement(Nat.0)
    forall(m: Nat) {
        if statement(m) {
            forall(d: Nat -> R, b: R) {
                coeff_quotient(d, b, m.suc.suc, m.suc) =
                    coeff_quotient(coeff_tail(d), b, m.suc, m)
                statement(m) = forall(e: Nat -> R, x: R) {
                    coeff_quotient(e, x, m.suc, m) = R.0
                }
                coeff_quotient(coeff_tail(d), b, m.suc, m) = R.0
                coeff_quotient(d, b, m.suc.suc, m.suc) = R.0
            }
            statement(m.suc)
        }
    }
    statement(Nat.0) and forall(m: Nat) {
        statement(m) implies statement(m.suc)
    }
    alt_induction(statement)
    forall(m: Nat) { statement(m) }
    statement(n)
    statement(n) = forall(d: Nat -> R, b: R) {
        coeff_quotient(d, b, n.suc, n) = R.0
    }
    coeff_quotient(p.coeff, a, n.suc, n) = R.0
    polynomial_quotient(p, a, n.suc).coeff(n) = R.0
}

/// At a valid support bound, a root recovers `p` as `(X - a) * q` where `q`
/// is the bundled synthetic quotient.
theorem polynomial_mul_linear_quotient_eq[R: CommRing](p: Polynomial[R], a: R, n: Nat) {
    polynomial_support_bounded_by(p, n) and polynomial_eval(p, a) = R.0 implies
    polynomial_mul(linear_polynomial(R.1, -a), polynomial_quotient(p, a, n.suc)) = p
} by {
    if polynomial_support_bounded_by(p, n) and polynomial_eval(p, a) = R.0 {
        forall(k: Nat) {
            if k = Nat.0 {
                polynomial_mul_x_minus_a_coeff_zero(a, polynomial_quotient(p, a, n.suc))
                polynomial_mul(linear_polynomial(R.1, -a), polynomial_quotient(p, a, n.suc)).coeff(Nat.0) =
                    (-a) * polynomial_quotient(p, a, n.suc).coeff(Nat.0)
                multiplicity_quotient_zero_coeff(p, a, n)
                polynomial_quotient(p, a, n.suc).coeff(Nat.0) = coeff_eval(coeff_tail(p.coeff), a, n)
                polynomial_mul(linear_polynomial(R.1, -a), polynomial_quotient(p, a, n.suc)).coeff(Nat.0) =
                    (-a) * coeff_eval(coeff_tail(p.coeff), a, n)
                lt_suc(n)
                n < n.suc
                lt_imp_lte[Nat](n, n.suc)
                n <= n.suc
                polynomial_support_bounded_by_monotone(p, n, n.suc)
                polynomial_support_bounded_by(p, n.suc)
                polynomial_eval_eq_eval_bound_of_support_bounded(p, a, n.suc)
                polynomial_eval(p, a) = polynomial_eval_bound(p, a, n.suc)
                polynomial_eval(p, a) = R.0
                polynomial_eval_bound(p, a, n.suc) = R.0
                polynomial_eval_bound_eq_coeff_eval(p, a, n.suc)
                polynomial_eval_bound(p, a, n.suc) = coeff_eval(p.coeff, a, n.suc)
                coeff_eval(p.coeff, a, n.suc) = R.0
                coeff_eval(p.coeff, a, n.suc) = p.coeff(Nat.0) + a * coeff_eval(coeff_tail(p.coeff), a, n)
                p.coeff(Nat.0) + a * coeff_eval(coeff_tail(p.coeff), a, n) = R.0
                p.coeff(Nat.0) + a * coeff_eval(coeff_tail(p.coeff), a, n) + -(a * coeff_eval(coeff_tail(p.coeff), a, n)) =
                    R.0 + -(a * coeff_eval(coeff_tail(p.coeff), a, n))
                a * coeff_eval(coeff_tail(p.coeff), a, n) + -(a * coeff_eval(coeff_tail(p.coeff), a, n)) = R.0
                p.coeff(Nat.0) + R.0 = p.coeff(Nat.0)
                R.0 + -(a * coeff_eval(coeff_tail(p.coeff), a, n)) = -(a * coeff_eval(coeff_tail(p.coeff), a, n))
                p.coeff(Nat.0) = -(a * coeff_eval(coeff_tail(p.coeff), a, n))
                mul_neg_left(a, coeff_eval(coeff_tail(p.coeff), a, n))
                (-a) * coeff_eval(coeff_tail(p.coeff), a, n) = -(a * coeff_eval(coeff_tail(p.coeff), a, n))
                p.coeff(Nat.0) = (-a) * coeff_eval(coeff_tail(p.coeff), a, n)
                polynomial_mul(linear_polynomial(R.1, -a), polynomial_quotient(p, a, n.suc)).coeff(Nat.0) =
                    p.coeff(Nat.0)
                k = Nat.0
                polynomial_mul(linear_polynomial(R.1, -a), polynomial_quotient(p, a, n.suc)).coeff(k) =
                    p.coeff(k)
            } else {
                k != Nat.0
                zero_or_suc(k)
                let kp: Nat satisfy {
                    k = kp.suc
                }
                k = kp.suc
                lte_suc_suc(Nat.0, kp)
                Nat.1 <= kp.suc
                kp.suc = k
                Nat.1 <= k
                if k < n.suc {
                    polynomial_mul_x_minus_a_coeff(a, polynomial_quotient(p, a, n.suc), k)
                    polynomial_mul(linear_polynomial(R.1, -a), polynomial_quotient(p, a, n.suc)).coeff(k) =
                        (-a) * polynomial_quotient(p, a, n.suc).coeff(k) +
                        polynomial_quotient(p, a, n.suc).coeff(k - Nat.1)
                    lt_cancel_suc(kp, n)
                    kp < n
                    multiplicity_quotient_recurrence(p, a, n, kp)
                    polynomial_quotient(p, a, n.suc).coeff(kp) =
                        p.coeff(kp.suc) + a * polynomial_quotient(p, a, n.suc).coeff(kp.suc)
                    kp.suc = k
                    polynomial_quotient(p, a, n.suc).coeff(k - Nat.1) =
                        polynomial_quotient(p, a, n.suc).coeff(kp)
                    polynomial_quotient(p, a, n.suc).coeff(kp.suc) =
                        polynomial_quotient(p, a, n.suc).coeff(k)
                    polynomial_quotient(p, a, n.suc).coeff(k - Nat.1) =
                        p.coeff(k) + a * polynomial_quotient(p, a, n.suc).coeff(k)
                    polynomial_mul(linear_polynomial(R.1, -a), polynomial_quotient(p, a, n.suc)).coeff(k) =
                        (-a) * polynomial_quotient(p, a, n.suc).coeff(k) +
                        (p.coeff(k) + a * polynomial_quotient(p, a, n.suc).coeff(k))
                    p.coeff(k) + a * polynomial_quotient(p, a, n.suc).coeff(k) =
                        a * polynomial_quotient(p, a, n.suc).coeff(k) + p.coeff(k)
                    (-a) * polynomial_quotient(p, a, n.suc).coeff(k) +
                        (p.coeff(k) + a * polynomial_quotient(p, a, n.suc).coeff(k)) =
                        (-a) * polynomial_quotient(p, a, n.suc).coeff(k) +
                        (a * polynomial_quotient(p, a, n.suc).coeff(k) + p.coeff(k))
                    (-a) * polynomial_quotient(p, a, n.suc).coeff(k) +
                        (a * polynomial_quotient(p, a, n.suc).coeff(k) + p.coeff(k)) =
                        ((-a) * polynomial_quotient(p, a, n.suc).coeff(k) +
                            a * polynomial_quotient(p, a, n.suc).coeff(k)) + p.coeff(k)
                    ((-a) * polynomial_quotient(p, a, n.suc).coeff(k) +
                        a * polynomial_quotient(p, a, n.suc).coeff(k)) + p.coeff(k) =
                        p.coeff(k) + ((-a) * polynomial_quotient(p, a, n.suc).coeff(k) +
                            a * polynomial_quotient(p, a, n.suc).coeff(k))
                    (-a) * polynomial_quotient(p, a, n.suc).coeff(k) +
                        a * polynomial_quotient(p, a, n.suc).coeff(k) =
                        ((-a) + a) * polynomial_quotient(p, a, n.suc).coeff(k)
                    (-a) + a = R.0
                    R.0 * polynomial_quotient(p, a, n.suc).coeff(k) = R.0
                    p.coeff(k) + R.0 = p.coeff(k)
                    polynomial_mul(linear_polynomial(R.1, -a), polynomial_quotient(p, a, n.suc)).coeff(k) =
                        p.coeff(k)
                } else {
                    not k < n.suc
                    polynomial_quotient_support_bounded_by(p, a, n.suc)
                    polynomial_support_bounded_by(polynomial_quotient(p, a, n.suc), n.suc)
                    polynomial_support_bounded_by_apply(polynomial_quotient(p, a, n.suc), n.suc, k)
                    polynomial_quotient(p, a, n.suc).coeff(k) = R.0
                    if k - Nat.1 < n.suc {
                        lt_imp_lte_suc(k - Nat.1, n.suc)
                        (k - Nat.1).suc <= n.suc
                        suc_sub_one(kp)
                        kp.suc - Nat.1 = kp
                        k = kp.suc
                        k - Nat.1 = kp
                        (k - Nat.1).suc = k
                        k <= n.suc
                        lt_or_lte(k, n.suc)
                        if k < n.suc {
                            false
                        }
                        n.suc <= k
                        lte_antisymm(k, n.suc)
                        k = n.suc
                        suc_sub_one(n)
                        n.suc - Nat.1 = n
                        k - Nat.1 = n
                        multiplicity_quotient_diagonal_zero(p, a, n)
                        polynomial_quotient(p, a, n.suc).coeff(n) = R.0
                        polynomial_quotient(p, a, n.suc).coeff(k - Nat.1) = R.0
                    } else {
                        not k - Nat.1 < n.suc
                        polynomial_support_bounded_by_apply(polynomial_quotient(p, a, n.suc), n.suc, k - Nat.1)
                        polynomial_quotient(p, a, n.suc).coeff(k - Nat.1) = R.0
                    }
                    polynomial_mul_x_minus_a_coeff(a, polynomial_quotient(p, a, n.suc), k)
                    polynomial_mul(linear_polynomial(R.1, -a), polynomial_quotient(p, a, n.suc)).coeff(k) =
                        (-a) * polynomial_quotient(p, a, n.suc).coeff(k) +
                        polynomial_quotient(p, a, n.suc).coeff(k - Nat.1)
                    polynomial_mul(linear_polynomial(R.1, -a), polynomial_quotient(p, a, n.suc)).coeff(k) =
                        (-a) * R.0 + R.0
                    (-a) * R.0 = R.0
                    polynomial_mul(linear_polynomial(R.1, -a), polynomial_quotient(p, a, n.suc)).coeff(k) =
                        R.0
                    polynomial_support_bounded_by_apply(p, n, k)
                    p.coeff(k) = R.0
                    polynomial_mul(linear_polynomial(R.1, -a), polynomial_quotient(p, a, n.suc)).coeff(k) =
                        p.coeff(k)
                }
            }
        }
        polynomial_ext_pointwise(p, polynomial_mul(linear_polynomial(R.1, -a),
            polynomial_quotient(p, a, n.suc)))
        p = polynomial_mul(linear_polynomial(R.1, -a), polynomial_quotient(p, a, n.suc))
    }
}

// ---------------------------------------------------------------------------
// The factor theorem at the coefficient level
// ---------------------------------------------------------------------------

/// A root of `p` gives the linear factor `X - a` as a polynomial witness for
/// divisibility: `(X - a) | p`.
theorem polynomial_root_imp_linear_divides[R: CommRing](p: Polynomial[R], a: R) {
    polynomial_eval(p, a) = R.0 implies polynomial_divides(linear_polynomial(R.1, -a), p)
} by {
    if polynomial_eval(p, a) = R.0 {
        polynomial_support_bound_exists(p)
        let n: Nat satisfy {
            polynomial_support_bounded_by(p, n)
        }
        polynomial_mul_linear_quotient_eq(p, a, n)
        polynomial_mul(linear_polynomial(R.1, -a), polynomial_quotient(p, a, n.suc)) = p
        exists(r: Polynomial[R]) {
            r = polynomial_quotient(p, a, n.suc) and
                polynomial_mul(linear_polynomial(R.1, -a), r) = p
        }
        polynomial_divides(linear_polynomial(R.1, -a), p)
    }
}

/// Divisibility by `X - a` forces `a` to be a root of `p`.
theorem polynomial_linear_divides_imp_root[R: CommRing](p: Polynomial[R], a: R) {
    polynomial_divides(linear_polynomial(R.1, -a), p) implies polynomial_eval(p, a) = R.0
} by {
    if polynomial_divides(linear_polynomial(R.1, -a), p) {
        let q: Polynomial[R] satisfy {
            polynomial_mul(linear_polynomial(R.1, -a), q) = p
        }
        polynomial_divides_eval_witness(linear_polynomial(R.1, -a), p, q, a)
        polynomial_eval(p, a) = polynomial_eval(linear_polynomial(R.1, -a), a) * polynomial_eval(q, a)
        linear_polynomial_neg_root(a)
        polynomial_eval(linear_polynomial(R.1, -a), a) = R.0
        polynomial_eval(p, a) = R.0 * polynomial_eval(q, a)
        R.0 * polynomial_eval(q, a) = R.0
        polynomial_eval(p, a) = R.0
    }
}

/// The linear factor theorem: `X - a` divides `p` exactly when `a` is a root
/// of `p`.
theorem polynomial_linear_divides_iff_root[R: CommRing](p: Polynomial[R], a: R) {
    polynomial_divides(linear_polynomial(R.1, -a), p) = (polynomial_eval(p, a) = R.0)
} by {
    if polynomial_divides(linear_polynomial(R.1, -a), p) {
        polynomial_linear_divides_imp_root(p, a)
        polynomial_eval(p, a) = R.0
    }
    if polynomial_eval(p, a) = R.0 {
        polynomial_root_imp_linear_divides(p, a)
        polynomial_divides(linear_polynomial(R.1, -a), p)
    }
}

// ---------------------------------------------------------------------------
// Root multiplicity
// ---------------------------------------------------------------------------

/// `a` is a root of `p` of multiplicity at least `k` when the `k`-th power of
/// `X - a` divides `p`.
define polynomial_root_multiplicity_at_least[R: CommRing](
    p: Polynomial[R], a: R, k: Nat
) -> Bool {
    polynomial_divides(polynomial_pow(linear_polynomial(R.1, -a), k), p)
}

/// A point is a root exactly when it is a root of multiplicity at least one.
theorem polynomial_root_multiplicity_one_iff_root[R: CommRing](p: Polynomial[R], a: R) {
    polynomial_root_multiplicity_at_least(p, a, Nat.1) = (polynomial_eval(p, a) = R.0)
} by {
    polynomial_pow_one(linear_polynomial(R.1, -a))
    polynomial_pow(linear_polynomial(R.1, -a), Nat.1) = linear_polynomial(R.1, -a)
    polynomial_root_multiplicity_at_least(p, a, Nat.1) =
        polynomial_divides(polynomial_pow(linear_polynomial(R.1, -a), Nat.1), p)
    polynomial_root_multiplicity_at_least(p, a, Nat.1) =
        polynomial_divides(linear_polynomial(R.1, -a), p)
    polynomial_linear_divides_iff_root(p, a)
    polynomial_divides(linear_polynomial(R.1, -a), p) = (polynomial_eval(p, a) = R.0)
    polynomial_root_multiplicity_at_least(p, a, Nat.1) = (polynomial_eval(p, a) = R.0)
}

/// Multiplicity is monotone: a root of multiplicity at least `k` has
/// multiplicity at least every `j <= k`.
theorem polynomial_root_multiplicity_at_least_monotone[R: CommRing](
    p: Polynomial[R], a: R, k: Nat, j: Nat
) {
    polynomial_root_multiplicity_at_least(p, a, k) and j <= k implies
        polynomial_root_multiplicity_at_least(p, a, j)
} by {
    if polynomial_root_multiplicity_at_least(p, a, k) and j <= k {
        polynomial_root_multiplicity_at_least(p, a, k) =
            polynomial_divides(polynomial_pow(linear_polynomial(R.1, -a), k), p)
        polynomial_divides(polynomial_pow(linear_polynomial(R.1, -a), k), p)
        let u: Polynomial[R] satisfy {
            polynomial_mul(polynomial_pow(linear_polynomial(R.1, -a), k), u) = p
        }
        add_sub(k, j)
        k - j + j = k
        j + (k - j) = k
        polynomial_pow_add(linear_polynomial(R.1, -a), j, k - j)
        polynomial_pow(linear_polynomial(R.1, -a), j + (k - j)) =
            polynomial_mul(polynomial_pow(linear_polynomial(R.1, -a), j),
                polynomial_pow(linear_polynomial(R.1, -a), k - j))
        polynomial_pow(linear_polynomial(R.1, -a), k) =
            polynomial_mul(polynomial_pow(linear_polynomial(R.1, -a), j),
                polynomial_pow(linear_polynomial(R.1, -a), k - j))
        polynomial_mul(polynomial_pow(linear_polynomial(R.1, -a), k), u) =
            polynomial_mul(polynomial_mul(polynomial_pow(linear_polynomial(R.1, -a), j),
                polynomial_pow(linear_polynomial(R.1, -a), k - j)), u)
        polynomial_mul_assoc(polynomial_pow(linear_polynomial(R.1, -a), j),
            polynomial_pow(linear_polynomial(R.1, -a), k - j), u)
        polynomial_mul(polynomial_mul(polynomial_pow(linear_polynomial(R.1, -a), j),
            polynomial_pow(linear_polynomial(R.1, -a), k - j)), u) =
            polynomial_mul(polynomial_pow(linear_polynomial(R.1, -a), j),
                polynomial_mul(polynomial_pow(linear_polynomial(R.1, -a), k - j), u))
        polynomial_mul(polynomial_pow(linear_polynomial(R.1, -a), k), u) =
            polynomial_mul(polynomial_pow(linear_polynomial(R.1, -a), j),
                polynomial_mul(polynomial_pow(linear_polynomial(R.1, -a), k - j), u))
        polynomial_mul(polynomial_pow(linear_polynomial(R.1, -a), k), u) = p
        p = polynomial_mul(polynomial_pow(linear_polynomial(R.1, -a), j),
            polynomial_mul(polynomial_pow(linear_polynomial(R.1, -a), k - j), u))
        exists(r: Polynomial[R]) {
            r = polynomial_mul(polynomial_pow(linear_polynomial(R.1, -a), k - j), u) and
                polynomial_mul(polynomial_pow(linear_polynomial(R.1, -a), j), r) = p
        }
        polynomial_divides(polynomial_pow(linear_polynomial(R.1, -a), j), p)
        polynomial_root_multiplicity_at_least(p, a, j) =
            polynomial_divides(polynomial_pow(linear_polynomial(R.1, -a), j), p)
        polynomial_root_multiplicity_at_least(p, a, j)
    }
}

/// `a` is a root of `p` of multiplicity exactly `k` when `(X - a)^k` divides
/// `p` but `(X - a)^(k + 1)` does not.
define polynomial_root_multiplicity_exact[R: CommRing](
    p: Polynomial[R], a: R, k: Nat
) -> Bool {
    polynomial_root_multiplicity_at_least(p, a, k) and
        not polynomial_root_multiplicity_at_least(p, a, k.suc)
}

/// A root of multiplicity exactly `k` has `(X - a)^k` dividing `p`.
theorem polynomial_root_multiplicity_exact_divides[R: CommRing](p: Polynomial[R], a: R, k: Nat) {
    polynomial_root_multiplicity_exact(p, a, k) implies
        polynomial_divides(polynomial_pow(linear_polynomial(R.1, -a), k), p)
} by {
    if polynomial_root_multiplicity_exact(p, a, k) {
        polynomial_root_multiplicity_exact(p, a, k) =
            (polynomial_root_multiplicity_at_least(p, a, k) and
                not polynomial_root_multiplicity_at_least(p, a, k.suc))
        polynomial_root_multiplicity_at_least(p, a, k)
        polynomial_root_multiplicity_at_least(p, a, k) =
            polynomial_divides(polynomial_pow(linear_polynomial(R.1, -a), k), p)
        polynomial_divides(polynomial_pow(linear_polynomial(R.1, -a), k), p)
    }
}

/// A root of multiplicity exactly `k` is not of multiplicity at least
/// `k + 1`: `(X - a)^(k + 1)` does not divide `p`.
theorem polynomial_root_multiplicity_exact_not_next[R: CommRing](p: Polynomial[R], a: R, k: Nat) {
    polynomial_root_multiplicity_exact(p, a, k) implies
        not polynomial_divides(polynomial_pow(linear_polynomial(R.1, -a), k.suc), p)
} by {
    if polynomial_root_multiplicity_exact(p, a, k) {
        polynomial_root_multiplicity_exact(p, a, k) =
            (polynomial_root_multiplicity_at_least(p, a, k) and
                not polynomial_root_multiplicity_at_least(p, a, k.suc))
        not polynomial_root_multiplicity_at_least(p, a, k.suc)
        polynomial_root_multiplicity_at_least(p, a, k.suc) =
            polynomial_divides(polynomial_pow(linear_polynomial(R.1, -a), k.suc), p)
        if polynomial_divides(polynomial_pow(linear_polynomial(R.1, -a), k.suc), p) {
            polynomial_divides(polynomial_pow(linear_polynomial(R.1, -a), k.suc), p) =
                polynomial_root_multiplicity_at_least(p, a, k.suc)
            polynomial_root_multiplicity_at_least(p, a, k.suc)
            false
        }
        not polynomial_divides(polynomial_pow(linear_polynomial(R.1, -a), k.suc), p)
    }
}

// ---------------------------------------------------------------------------
// Distinct roots: the product of the linear factors divides, and the degree
// bound
// ---------------------------------------------------------------------------

/// The product of the linear factors `X - r` of the roots in a list.
define linear_factor_product[F: Field](roots: List[F]) -> Polynomial[F] {
    match roots {
        List.nil {
            Polynomial[F].one
        }
        List.cons(head, tail) {
            polynomial_mul(linear_polynomial(F.1, -head), linear_factor_product(tail))
        }
    }
}

/// A root of the first factor of a product is a root of the product.
theorem multiplicity_root_mul_left[R: CommRing](p: Polynomial[R], q: Polynomial[R], a: R) {
    polynomial_eval(p, a) = R.0 implies polynomial_eval(polynomial_mul(p, q), a) = R.0
} by {
    if polynomial_eval(p, a) = R.0 {
        polynomial_eval_mul(p, q, a)
        polynomial_eval(polynomial_mul(p, q), a) = polynomial_eval(p, a) * polynomial_eval(q, a)
        polynomial_eval(p, a) = R.0
        polynomial_eval(polynomial_mul(p, q), a) = R.0 * polynomial_eval(q, a)
        R.0 * polynomial_eval(q, a) = R.0
        polynomial_eval(polynomial_mul(p, q), a) = R.0
    }
}

/// Two distinct roots force divisibility by the product of the two linear
/// factors.
theorem polynomial_two_roots_product_divides[F: Field](p: Polynomial[F], a: F, b: F) {
    a != b and polynomial_eval(p, a) = F.0 and polynomial_eval(p, b) = F.0 implies
    polynomial_divides(polynomial_mul(linear_polynomial(F.1, -a),
        linear_polynomial(F.1, -b)), p)
} by {
    if a != b and polynomial_eval(p, a) = F.0 and polynomial_eval(p, b) = F.0 {
        polynomial_root_imp_linear_divides(p, a)
        polynomial_divides(linear_polynomial(F.1, -a), p)
        let q: Polynomial[F] satisfy {
            polynomial_mul(linear_polynomial(F.1, -a), q) = p
        }
        polynomial_divides_eval_witness(linear_polynomial(F.1, -a), p, q, b)
        polynomial_eval(p, b) = polynomial_eval(linear_polynomial(F.1, -a), b) * polynomial_eval(q, b)
        polynomial_eval(p, b) = F.0
        polynomial_eval(linear_polynomial(F.1, -a), b) * polynomial_eval(q, b) = F.0
        linear_polynomial_eval(F.1, -a, b)
        polynomial_eval(linear_polynomial(F.1, -a), b) = -a + F.1 * b
        F.1 * b = b
        polynomial_eval(linear_polynomial(F.1, -a), b) = -a + b
        -a + b = b + -a
        b + -a = b - a
        polynomial_eval(linear_polynomial(F.1, -a), b) = b - a
        (b - a) * polynomial_eval(q, b) = F.0
        if b - a = F.0 {
            polynomial_sub_eq_zero_iff(b, a)
            (b - a = F.0) = (b = a)
            b = a
            false
        }
        b - a != F.0
        field_mul_eq_zero(b - a, polynomial_eval(q, b))
        polynomial_eval(q, b) = F.0
        polynomial_root_imp_linear_divides(q, b)
        polynomial_divides(linear_polynomial(F.1, -b), q)
        let r: Polynomial[F] satisfy {
            polynomial_mul(linear_polynomial(F.1, -b), r) = q
        }
        polynomial_mul_assoc(linear_polynomial(F.1, -a), linear_polynomial(F.1, -b), r)
        polynomial_mul(polynomial_mul(linear_polynomial(F.1, -a), linear_polynomial(F.1, -b)), r) =
            polynomial_mul(linear_polynomial(F.1, -a), polynomial_mul(linear_polynomial(F.1, -b), r))
        polynomial_mul(linear_polynomial(F.1, -b), r) = q
        polynomial_mul(linear_polynomial(F.1, -a), q) = p
        polynomial_mul(polynomial_mul(linear_polynomial(F.1, -a), linear_polynomial(F.1, -b)), r) = p
        exists(w: Polynomial[F]) {
            w = r and polynomial_mul(polynomial_mul(linear_polynomial(F.1, -a),
                linear_polynomial(F.1, -b)), w) = p
        }
        polynomial_divides(polynomial_mul(linear_polynomial(F.1, -a),
            linear_polynomial(F.1, -b)), p)
    }
}

/// A linear factor not vanishing at `a` is coprime to any divisor of a
/// polynomial vanishing at `a`: if `f | p`, `X - a | p` and `f(a) != 0` then
/// `(X - a) * f | p`.
theorem polynomial_coprime_linear_product_divides[F: Field](
    f: Polynomial[F], a: F, p: Polynomial[F]
) {
    polynomial_divides(f, p) and polynomial_divides(linear_polynomial(F.1, -a), p) and
    polynomial_eval(f, a) != F.0
    implies polynomial_divides(polynomial_mul(linear_polynomial(F.1, -a), f), p)
} by {
    if polynomial_divides(f, p) and polynomial_divides(linear_polynomial(F.1, -a), p) and
        polynomial_eval(f, a) != F.0 {
        let q: Polynomial[F] satisfy {
            polynomial_mul(f, q) = p
        }
        polynomial_divides_eval_witness(f, p, q, a)
        polynomial_eval(p, a) = polynomial_eval(f, a) * polynomial_eval(q, a)
        polynomial_divides_eval_witness(linear_polynomial(F.1, -a), p, polynomial_constant(F.1), a)
        let w: Polynomial[F] satisfy {
            polynomial_mul(linear_polynomial(F.1, -a), w) = p
        }
        polynomial_divides_eval_witness(linear_polynomial(F.1, -a), p, w, a)
        polynomial_eval(p, a) = polynomial_eval(linear_polynomial(F.1, -a), a) * polynomial_eval(w, a)
        linear_polynomial_neg_root(a)
        polynomial_eval(linear_polynomial(F.1, -a), a) = F.0
        polynomial_eval(p, a) = F.0 * polynomial_eval(w, a)
        F.0 * polynomial_eval(w, a) = F.0
        polynomial_eval(p, a) = F.0
        polynomial_eval(f, a) * polynomial_eval(q, a) = F.0
        field_mul_eq_zero(polynomial_eval(f, a), polynomial_eval(q, a))
        if polynomial_eval(f, a) = F.0 {
            false
        }
        polynomial_eval(q, a) = F.0
        polynomial_root_imp_linear_divides(q, a)
        polynomial_divides(linear_polynomial(F.1, -a), q)
        let r: Polynomial[F] satisfy {
            polynomial_mul(linear_polynomial(F.1, -a), r) = q
        }
        polynomial_mul_assoc(f, linear_polynomial(F.1, -a), r)
        polynomial_mul(polynomial_mul(f, linear_polynomial(F.1, -a)), r) =
            polynomial_mul(f, polynomial_mul(linear_polynomial(F.1, -a), r))
        polynomial_mul(linear_polynomial(F.1, -a), r) = q
        polynomial_mul(f, q) = p
        polynomial_mul(polynomial_mul(f, linear_polynomial(F.1, -a)), r) = p
        polynomial_mul_comm(f, linear_polynomial(F.1, -a))
        polynomial_mul(f, linear_polynomial(F.1, -a)) =
            polynomial_mul(linear_polynomial(F.1, -a), f)
        polynomial_mul(polynomial_mul(linear_polynomial(F.1, -a), f), r) = p
        exists(s: Polynomial[F]) {
            s = r and polynomial_mul(polynomial_mul(linear_polynomial(F.1, -a), f), s) = p
        }
        polynomial_divides(polynomial_mul(linear_polynomial(F.1, -a), f), p)
    }
}


/// A list contains everything its tail contains.
lemma multiplicity_cons_contains_right[T](head: T, tail: List[T], item: T) {
    tail.contains(item) implies List.cons(head, tail).contains(item)
} by {
    if tail.contains(item) {
        List.cons(head, tail).contains(item) =
            (if head = item { true } else { tail.contains(item) })
        if head = item {
            List.cons(head, tail).contains(item) = true
            List.cons(head, tail).contains(item)
        } else {
            List.cons(head, tail).contains(item) = tail.contains(item)
            List.cons(head, tail).contains(item)
        }
    }
}

/// A cons list contains its head.
lemma multiplicity_cons_contains_head[T](head: T, tail: List[T]) {
    List.cons(head, tail).contains(head)
} by {
    List.cons(head, tail).contains(head) =
        (if head = head { true } else { tail.contains(head) })
    List.cons(head, tail).contains(head) = true
    List.cons(head, tail).contains(head)
}

/// A list avoiding an item has a tail avoiding it.
lemma multiplicity_not_cons_contains_not_tail[T](head: T, tail: List[T], item: T) {
    not List.cons(head, tail).contains(item) implies not tail.contains(item)
} by {
    if not List.cons(head, tail).contains(item) {
        if tail.contains(item) {
            multiplicity_cons_contains_right(head, tail, item)
            List.cons(head, tail).contains(item)
            false
        }
        not tail.contains(item)
    }
}

/// A unique cons list does not contain its head again in the tail.
lemma multiplicity_cons_unique_not_contains[T](head: T, tail: List[T]) {
    List.cons(head, tail).is_unique implies not tail.contains(head)
} by {
    if List.cons(head, tail).is_unique {
        if tail.contains(head) {
            List.cons(head, tail).unique = List.cons(head, tail)
            List.cons(head, tail).unique =
                (if tail.contains(head) { tail.unique } else { List.cons(head, tail.unique) })
            List.cons(head, tail).unique = tail.unique
            tail.unique = List.cons(head, tail)
            unique_length(tail)
            tail.unique.length <= tail.length
            List.cons(head, tail).length = tail.length.suc
            tail.unique.length = tail.length.suc
            tail.length.suc <= tail.length
            false
        }
        not tail.contains(head)
    }
}

/// Evaluating the product of linear factors at a point outside the list is a
/// product of nonzero differences, hence nonzero.
theorem linear_factor_product_eval_ne_zero[F: Field](roots: List[F], a: F) {
    roots.is_unique and not roots.contains(a) implies
    polynomial_eval(linear_factor_product(roots), a) != F.0
} by {
    define q(items: List[F], x: F) -> Bool {
        items.is_unique and not items.contains(x) implies
        polynomial_eval(linear_factor_product(items), x) != F.0
    }

    if List.nil[F].is_unique and not List.nil[F].contains(a) {
        linear_factor_product(List.nil[F]) = Polynomial[F].one
        polynomial_eval_constant(F.1, a)
        polynomial_eval(Polynomial[F].constant(F.1), a) = F.1
        polynomial_eval(linear_factor_product(List.nil[F]), a) = F.1
        F.1 != F.0
        polynomial_eval(linear_factor_product(List.nil[F]), a) != F.0
    }
    q(List.nil[F], a) = (List.nil[F].is_unique and not List.nil[F].contains(a) implies
        polynomial_eval(linear_factor_product(List.nil[F]), a) != F.0)
    q(List.nil[F], a)

    forall(head: F, tail: List[F]) {
        if q(tail, a) {
            if List.cons(head, tail).is_unique and not List.cons(head, tail).contains(a) {
                if a = head {
                    multiplicity_cons_contains_head(head, tail)
                    List.cons(head, tail).contains(head)
                    a = head
                    List.cons(head, tail).contains(a)
                    not List.cons(head, tail).contains(a)
                    false
                }
                a != head
                if a - head = F.0 {
                    polynomial_sub_eq_zero_iff(a, head)
                    (a - head = F.0) = (a = head)
                    a = head
                    false
                }
                a - head != F.0
                unique_implies_tail_unique(head, tail)
                tail.is_unique
                multiplicity_not_cons_contains_not_tail(head, tail, a)
                not tail.contains(a)
                q(tail, a) = (tail.is_unique and not tail.contains(a) implies
                    polynomial_eval(linear_factor_product(tail), a) != F.0)
                polynomial_eval(linear_factor_product(tail), a) != F.0
                linear_factor_product(List.cons(head, tail)) =
                    polynomial_mul(linear_polynomial(F.1, -head), linear_factor_product(tail))
                polynomial_eval_mul(linear_polynomial(F.1, -head), linear_factor_product(tail), a)
                polynomial_eval(polynomial_mul(linear_polynomial(F.1, -head),
                    linear_factor_product(tail)), a) =
                    polynomial_eval(linear_polynomial(F.1, -head), a) *
                    polynomial_eval(linear_factor_product(tail), a)
                linear_polynomial_eval(F.1, -head, a)
                polynomial_eval(linear_polynomial(F.1, -head), a) = -head + F.1 * a
                F.1 * a = a
                polynomial_eval(linear_polynomial(F.1, -head), a) = -head + a
                -head + a = a + -head
                a + -head = a - head
                polynomial_eval(linear_polynomial(F.1, -head), a) = a - head
                polynomial_eval(polynomial_mul(linear_polynomial(F.1, -head),
                    linear_factor_product(tail)), a) = (a - head) * polynomial_eval(linear_factor_product(tail), a)
                field_mul_nonzero(a - head, polynomial_eval(linear_factor_product(tail), a))
                (a - head) * polynomial_eval(linear_factor_product(tail), a) != F.0
                polynomial_eval(polynomial_mul(linear_polynomial(F.1, -head),
                    linear_factor_product(tail)), a) != F.0
                polynomial_eval(linear_factor_product(List.cons(head, tail)), a) != F.0
            }
            q(List.cons(head, tail), a)
        }
    }

    List.induction(function(items: List[F]) { q(items, a) })
    forall(items: List[F]) { q(items, a) }
    q(roots, a)
    q(roots, a) = (roots.is_unique and not roots.contains(a) implies
        polynomial_eval(linear_factor_product(roots), a) != F.0)
    roots.is_unique and not roots.contains(a)
    polynomial_eval(linear_factor_product(roots), a) != F.0
}

/// The product of the linear factors of all roots of `p` divides `p`.
theorem polynomial_root_list_product_divides[F: Field](p: Polynomial[F], roots: List[F]) {
    roots.is_unique and polynomial_roots_on_list(p, roots) implies
    polynomial_divides(linear_factor_product(roots), p)
} by {
    define q(items: List[F]) -> Bool {
        items.is_unique and polynomial_roots_on_list(p, items) implies
        polynomial_divides(linear_factor_product(items), p)
    }

    q(List.nil[F]) = (List.nil[F].is_unique and polynomial_roots_on_list(p, List.nil[F]) implies
        polynomial_divides(linear_factor_product(List.nil[F]), p))
    if List.nil[F].is_unique and polynomial_roots_on_list(p, List.nil[F]) {
        linear_factor_product(List.nil[F]) = Polynomial[F].one
        polynomial_mul_one_left(p)
        polynomial_mul(Polynomial[F].one, p) = p
        exists(r: Polynomial[F]) {
            r = p and polynomial_mul(linear_factor_product(List.nil[F]), r) = p
        }
        polynomial_divides(linear_factor_product(List.nil[F]), p)
    }
    q(List.nil[F])

    forall(head: F, tail: List[F]) {
        if q(tail) {
            if List.cons(head, tail).is_unique and
                polynomial_roots_on_list(p, List.cons(head, tail)) {
                polynomial_roots_on_list(p, List.cons(head, tail)) = forall(x: F) {
                    List.cons(head, tail).contains(x) implies polynomial_eval(p, x) = F.0
                }
                multiplicity_cons_contains_head(head, tail)
                List.cons(head, tail).contains(head)
                polynomial_eval(p, head) = F.0
                polynomial_root_imp_linear_divides(p, head)
                polynomial_divides(linear_polynomial(F.1, -head), p)
                unique_implies_tail_unique(head, tail)
                tail.is_unique
                forall(x: F) {
                    if tail.contains(x) {
                        multiplicity_cons_contains_right(head, tail, x)
                        List.cons(head, tail).contains(x)
                        polynomial_eval(p, x) = F.0
                    }
                }
                polynomial_roots_on_list(p, tail) = forall(x: F) {
                    tail.contains(x) implies polynomial_eval(p, x) = F.0
                }
                polynomial_roots_on_list(p, tail)
                q(tail) = (tail.is_unique and polynomial_roots_on_list(p, tail) implies
                    polynomial_divides(linear_factor_product(tail), p))
                polynomial_divides(linear_factor_product(tail), p)
                multiplicity_cons_unique_not_contains(head, tail)
                not tail.contains(head)
                linear_factor_product_eval_ne_zero(tail, head)
                polynomial_eval(linear_factor_product(tail), head) != F.0
                polynomial_coprime_linear_product_divides(linear_factor_product(tail), head, p)
                polynomial_divides(polynomial_mul(linear_polynomial(F.1, -head),
                    linear_factor_product(tail)), p)
                linear_factor_product(List.cons(head, tail)) =
                    polynomial_mul(linear_polynomial(F.1, -head), linear_factor_product(tail))
                polynomial_divides(linear_factor_product(List.cons(head, tail)), p)
                q(List.cons(head, tail)) = (List.cons(head, tail).is_unique and
                    polynomial_roots_on_list(p, List.cons(head, tail)) implies
                    polynomial_divides(linear_factor_product(List.cons(head, tail)), p))
                q(List.cons(head, tail))
            }
            q(List.cons(head, tail))
        }
    }

    List.induction(function(items: List[F]) { q(items) })
    forall(items: List[F]) { q(items) }
    q(roots)
    q(roots) = (roots.is_unique and polynomial_roots_on_list(p, roots) implies
        polynomial_divides(linear_factor_product(roots), p))
    roots.is_unique and polynomial_roots_on_list(p, roots)
    polynomial_divides(linear_factor_product(roots), p)
}

/// A nonzero polynomial of degree at most `d` has at most `d` distinct roots.
theorem polynomial_root_count_le_degree[F: Field](p: Polynomial[F], roots: List[F], d: Nat) {
    polynomial_degree_le(p, d) and p != Polynomial[F].zero and roots.is_unique and
        polynomial_roots_on_list(p, roots)
    implies roots.length <= d
} by {
    if polynomial_degree_le(p, d) and p != Polynomial[F].zero and roots.is_unique and
        polynomial_roots_on_list(p, roots) {
        polynomial_degree_le(p, d) = polynomial_support_bounded_by(p, d.suc)
        polynomial_support_bounded_by(p, d.suc)
        polynomial_root_list_length_lt_support_bound(p, roots, d.suc)
        roots.length < d.suc
        if roots.length = Nat.0 {
            lt_imp_lte[Nat](Nat.0, d)
            Nat.0 <= d
            roots.length <= d
        } else {
            zero_or_suc(roots.length)
            let lp: Nat satisfy {
                roots.length = lp.suc
            }
            roots.length = lp.suc
            lt_cancel_suc(lp, d)
            lp < d
            lt_imp_lte_suc(lp, d)
            lp.suc <= d
            roots.length <= d
        }
    }
}
