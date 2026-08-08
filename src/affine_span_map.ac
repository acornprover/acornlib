/// Affine spans and image-preimage comparison under affine maps.

from algebra.add_comm_group import AddCommGroup
from affine_subspace import AffineSubspace, affine_subspace_ext, affine_subspace_subset,
    affine_span, affine_span_space, affine_span_contains_src, affine_span_subset
from affine_map import AffineMap, affine_map_image, affine_map_image_contains,
    affine_map_image_some, affine_map_image_space, affine_map_image_contains_iff,
    affine_map_image_contains_apply, affine_map_preimage, affine_map_preimage_some,
    affine_map_preimage_space, affine_map_preimage_contains_iff

/// True if `y` is the image under an affine map of a source point satisfying `src`.
define affine_map_set_image_contains[V1: AddCommGroup, P1, V2: AddCommGroup, P2](
    f: AffineMap[V1, P1, V2, P2], src: P1 -> Bool, y: P2) -> Bool {
    exists(x: P1) {
        src(x) and f.to_fun(x) = y
    }
}

/// A source point satisfying the predicate maps into the point-image predicate.
theorem affine_map_set_image_contains_apply[V1: AddCommGroup, P1, V2: AddCommGroup, P2](
    f: AffineMap[V1, P1, V2, P2], src: P1 -> Bool, x: P1) {
    src(x) implies affine_map_set_image_contains(f, src, f.to_fun(x))
} by {
    if src(x) {
        affine_map_set_image_contains(f, src, f.to_fun(x))
    }
}

/// The span of the pointwise image of a predicate is contained in the image of the source span.
theorem affine_span_image_subset_image_span[V1: AddCommGroup, P1, V2: AddCommGroup, P2](
    f: AffineMap[V1, P1, V2, P2], src: P1 -> Bool, img: AffineSubspace[V2, P2]) {
    affine_map_image(f, affine_span(f.src, src)) = Option.some(img) implies
        affine_subspace_subset(
            affine_span(f.dst, affine_map_set_image_contains(f, src)), img)
} by {
    if affine_map_image(f, affine_span(f.src, src)) = Option.some(img) {
        affine_map_image_space(f, affine_span(f.src, src), img)
        forall(y: P2) {
            if affine_map_set_image_contains(f, src, y) {
                let x: P1 satisfy {
                    src(x) and f.to_fun(x) = y
                }
                affine_span_contains_src(f.src, src, x)
                affine_map_image_contains_apply(f, affine_span(f.src, src), img, x)
                img.contains(f.to_fun(x))
                img.contains(y)
            }
        }
        affine_span_subset(f.dst, affine_map_set_image_contains(f, src), img)
        affine_subspace_subset(affine_span(f.dst, affine_map_set_image_contains(f, src)), img)
    }
}

/// The preimage of the target image span contains each source generator.
theorem affine_span_image_preimage_contains_src[V1: AddCommGroup, P1, V2: AddCommGroup, P2](
    f: AffineMap[V1, P1, V2, P2], src: P1 -> Bool, pre: AffineSubspace[V1, P1]) {
    affine_map_preimage(f, affine_span(f.dst, affine_map_set_image_contains(f, src))) = Option.some(pre)
        implies forall(x: P1) {
            src(x) implies pre.contains(x)
        }
} by {
    if affine_map_preimage(f, affine_span(f.dst, affine_map_set_image_contains(f, src))) = Option.some(pre) {
        let target = affine_span(f.dst, affine_map_set_image_contains(f, src))
        forall(x: P1) {
            if src(x) {
                affine_map_set_image_contains_apply(f, src, x)
                affine_span_contains_src(f.dst, affine_map_set_image_contains(f, src), f.to_fun(x))
                target.contains(f.to_fun(x))
                affine_map_preimage_contains_iff(f, target, pre, x)
                pre.contains(x)
            }
        }
    }
}

/// The image of the source span is contained in the span of the pointwise image.
theorem affine_map_image_span_subset_span_image[V1: AddCommGroup, P1, V2: AddCommGroup, P2](
    f: AffineMap[V1, P1, V2, P2], src: P1 -> Bool, img: AffineSubspace[V2, P2]) {
    affine_map_image(f, affine_span(f.src, src)) = Option.some(img) implies
        affine_subspace_subset(img,
            affine_span(f.dst, affine_map_set_image_contains(f, src)))
} by {
    let source_span = affine_span(f.src, src)
    let target_span = affine_span(f.dst, affine_map_set_image_contains(f, src))
    if affine_map_image(f, source_span) = Option.some(img) {
        affine_span_space(f.dst, affine_map_set_image_contains(f, src))
        affine_map_preimage_some(f, target_span)
        let pre: AffineSubspace[V1, P1] satisfy {
            affine_map_preimage(f, target_span) = Option.some(pre)
        }
        affine_map_preimage_space(f, target_span, pre)
        affine_span_image_preimage_contains_src(f, src, pre)
        forall(x: P1) {
            src(x) implies pre.contains(x)
        }
        affine_span_subset(f.src, src, pre)
        affine_subspace_subset(source_span, pre)
        affine_subspace_subset(source_span, pre) = forall(x: P1) {
            source_span.contains(x) implies pre.contains(x)
        }
        forall(y: P2) {
            if img.contains(y) {
                affine_map_image_contains_iff(f, source_span, img, y)
                let x: P1 satisfy {
                    source_span.contains(x) and f.to_fun(x) = y
                }
                affine_map_preimage_contains_iff(f, target_span, pre, x)
                target_span.contains(f.to_fun(x))
                target_span.contains(y)
            }
        }
        affine_subspace_subset(img, target_span)
    }
}

/// The image of the source span and the span of the pointwise image have the same points.
theorem affine_map_image_affine_span_contains_eq[V1: AddCommGroup, P1, V2: AddCommGroup, P2](
    f: AffineMap[V1, P1, V2, P2], src: P1 -> Bool, img: AffineSubspace[V2, P2]) {
    affine_map_image(f, affine_span(f.src, src)) = Option.some(img) implies
        forall(y: P2) {
            img.contains(y) = affine_span(f.dst, affine_map_set_image_contains(f, src)).contains(y)
        }
} by {
    if affine_map_image(f, affine_span(f.src, src)) = Option.some(img) {
        affine_span_image_subset_image_span(f, src, img)
        affine_map_image_span_subset_span_image(f, src, img)
        affine_subspace_subset(affine_span(f.dst, affine_map_set_image_contains(f, src)), img) = forall(y: P2) {
            affine_span(f.dst, affine_map_set_image_contains(f, src)).contains(y) implies img.contains(y)
        }
        affine_subspace_subset(img, affine_span(f.dst, affine_map_set_image_contains(f, src))) = forall(y: P2) {
            img.contains(y) implies affine_span(f.dst, affine_map_set_image_contains(f, src)).contains(y)
        }
        forall(y: P2) {
            if img.contains(y) {
                affine_span(f.dst, affine_map_set_image_contains(f, src)).contains(y)
            }
            if affine_span(f.dst, affine_map_set_image_contains(f, src)).contains(y) {
                img.contains(y)
            }
            img.contains(y) = affine_span(f.dst, affine_map_set_image_contains(f, src)).contains(y)
        }
    }
}

/// The image of the source span is exactly the span of the pointwise image.
theorem affine_map_image_affine_span_eq[V1: AddCommGroup, P1, V2: AddCommGroup, P2](
    f: AffineMap[V1, P1, V2, P2], src: P1 -> Bool, img: AffineSubspace[V2, P2]) {
    affine_map_image(f, affine_span(f.src, src)) = Option.some(img) implies
        img = affine_span(f.dst, affine_map_set_image_contains(f, src))
} by {
    let target = affine_span(f.dst, affine_map_set_image_contains(f, src))
    if affine_map_image(f, affine_span(f.src, src)) = Option.some(img) {
        affine_map_image_space(f, affine_span(f.src, src), img)
        affine_span_space(f.dst, affine_map_set_image_contains(f, src))
        img.space = target.space
        affine_map_image_affine_span_contains_eq(f, src, img)
        forall(y: P2) {
            img.contains(y) = target.contains(y)
        }
        affine_subspace_ext(img, target)
        img = target
    }
}

/// The bundled image of the source span is the target span of the pointwise image.
theorem affine_map_image_affine_span_eq_some[V1: AddCommGroup, P1, V2: AddCommGroup, P2](
    f: AffineMap[V1, P1, V2, P2], src: P1 -> Bool) {
    affine_map_image(f, affine_span(f.src, src)) =
        Option.some(affine_span(f.dst, affine_map_set_image_contains(f, src)))
} by {
    let source_span = affine_span(f.src, src)
    let target_span = affine_span(f.dst, affine_map_set_image_contains(f, src))
    affine_span_space(f.src, src)
    affine_map_image_some(f, source_span)
    let img: AffineSubspace[V2, P2] satisfy {
        affine_map_image(f, source_span) = Option.some(img)
    }
    affine_map_image_affine_span_eq(f, src, img)
    img = target_span
}

/// Membership in the target span is equivalent to membership in the image of the source span.
theorem affine_map_image_affine_span_contains_iff[V1: AddCommGroup, P1, V2: AddCommGroup, P2](
    f: AffineMap[V1, P1, V2, P2], src: P1 -> Bool, y: P2) {
    affine_span(f.dst, affine_map_set_image_contains(f, src)).contains(y) =
        affine_map_image_contains(f, affine_span(f.src, src), y)
} by {
    let source_span = affine_span(f.src, src)
    let target_span = affine_span(f.dst, affine_map_set_image_contains(f, src))
    affine_map_image_affine_span_eq_some(f, src)
    affine_map_image_contains_iff(f, source_span, target_span, y)
}

/// Applying an affine map to any point of the source span lands in the span of the pointwise image.
theorem affine_map_image_affine_span_contains_apply[V1: AddCommGroup, P1, V2: AddCommGroup, P2](
    f: AffineMap[V1, P1, V2, P2], src: P1 -> Bool, x: P1) {
    affine_span(f.src, src).contains(x) implies
        affine_span(f.dst, affine_map_set_image_contains(f, src)).contains(f.to_fun(x))
} by {
    let source_span = affine_span(f.src, src)
    let target_span = affine_span(f.dst, affine_map_set_image_contains(f, src))
    if source_span.contains(x) {
        affine_map_image_affine_span_eq_some(f, src)
        affine_map_image_contains_apply(f, source_span, target_span, x)
        target_span.contains(f.to_fun(x))
    }
}

/// Any point in the span of the pointwise image has a preimage in the source span.
theorem affine_map_image_affine_span_contains_witness[V1: AddCommGroup, P1, V2: AddCommGroup, P2](
    f: AffineMap[V1, P1, V2, P2], src: P1 -> Bool, y: P2) {
    affine_span(f.dst, affine_map_set_image_contains(f, src)).contains(y) implies
        exists(x: P1) {
            affine_span(f.src, src).contains(x) and f.to_fun(x) = y
        }
} by {
    let source_span = affine_span(f.src, src)
    let target_span = affine_span(f.dst, affine_map_set_image_contains(f, src))
    if target_span.contains(y) {
        affine_map_image_affine_span_contains_iff(f, src, y)
        let x: P1 satisfy {
            source_span.contains(x) and f.to_fun(x) = y
        }
        exists(z: P1) {
            source_span.contains(z) and f.to_fun(z) = y
        }
    }
}

/// The preimage of the target image span exists.
theorem affine_map_preimage_affine_span_image_some[V1: AddCommGroup, P1, V2: AddCommGroup, P2](
    f: AffineMap[V1, P1, V2, P2], src: P1 -> Bool) {
    exists(pre: AffineSubspace[V1, P1]) {
        affine_map_preimage(f, affine_span(f.dst, affine_map_set_image_contains(f, src))) =
            Option.some(pre)
    }
} by {
    let target_span = affine_span(f.dst, affine_map_set_image_contains(f, src))
    affine_span_space(f.dst, affine_map_set_image_contains(f, src))
    affine_map_preimage_some(f, target_span)
}

/// The source span is contained in the preimage of the target image span.
theorem affine_span_subset_preimage_affine_span_image[V1: AddCommGroup, P1, V2: AddCommGroup, P2](
    f: AffineMap[V1, P1, V2, P2], src: P1 -> Bool, pre: AffineSubspace[V1, P1]) {
    affine_map_preimage(f, affine_span(f.dst, affine_map_set_image_contains(f, src))) =
        Option.some(pre) implies affine_subspace_subset(affine_span(f.src, src), pre)
} by {
    let target_span = affine_span(f.dst, affine_map_set_image_contains(f, src))
    if affine_map_preimage(f, target_span) = Option.some(pre) {
        affine_map_preimage_space(f, target_span, pre)
        affine_span_image_preimage_contains_src(f, src, pre)
        forall(x: P1) {
            src(x) implies pre.contains(x)
        }
        affine_span_subset(f.src, src, pre)
        affine_subspace_subset(affine_span(f.src, src), pre)
    }
}
