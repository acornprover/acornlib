from nat import Nat
from int import Int, exp_one, exp_zero, exp_add
from polynomial import Polynomial, coeff_tail, coeff_eval
from data.int.int_coprime import is_coprime, is_coprime_symm, divides_of_coprime_pow

numerals Int

/// The homogeneous value of the first `n` coefficients at a fraction with numerator `a` and
/// denominator `b`.
///
/// This is the value at `a / b` cleared of denominators: the sum of `c(i) * a^i * b^(n - 1 - i)`.
/// Written this way it needs no rationals at all, which is what makes the divisibility argument
/// available over the integers.
///
/// Horner's rule in the same shape as `coeff_eval`: the constant coefficient carries the whole
/// power of the denominator, and everything above it carries a factor of the numerator.
define coeff_hom_eval(c: Nat -> Int, a: Int, b: Int, n: Nat) -> Int {
    match n {
        Nat.zero {
            Int.0
        }
        Nat.suc(pred) {
            c(Nat.0) * b.pow(pred) + a * coeff_hom_eval(coeff_tail(c), a, b, pred)
        }
    }
}

/// The homogeneous value at a successor bound splits off the constant coefficient.
theorem coeff_hom_eval_suc(c: Nat -> Int, a: Int, b: Int, n: Nat) {
    coeff_hom_eval(c, a, b, n.suc)
        = c(Nat.0) * b.pow(n) + a * coeff_hom_eval(coeff_tail(c), a, b, n)
}

/// The homogeneous value at bound zero vanishes.
theorem coeff_hom_eval_zero(c: Nat -> Int, a: Int, b: Int) {
    coeff_hom_eval(c, a, b, Nat.0) = Int.0
}

/// With denominator one, the homogeneous value is the ordinary value.
///
/// The sanity link to `coeff_eval`: a fraction with denominator one is an integer, and the
/// cleared equation is then the equation itself. So the numerator statement below contains the
/// integer root test.
theorem coeff_hom_eval_one(c: Nat -> Int, a: Int, n: Nat) {
    coeff_hom_eval(c, a, Int.1, n) = coeff_eval(c, a, n)
} by {
    define w(k: Nat) -> Bool {
        forall(d: Nat -> Int) {
            coeff_hom_eval(d, a, Int.1, k) = coeff_eval(d, a, k)
        }
    }
    forall(d: Nat -> Int) {
        coeff_hom_eval_zero(d, a, Int.1)
        (coeff_eval(d, a, Nat.zero) = Int.0)
        (Nat.zero = Nat.0)
        (coeff_eval(d, a, Nat.0) = Int.0)
        coeff_hom_eval(d, a, Int.1, Nat.0) = coeff_eval(d, a, Nat.0)
    }
    w(Nat.0)
    forall(k: Nat) {
        if w(k) {
            forall(d: Nat -> Int) {
                (forall(e: Nat -> Int) {
                    coeff_hom_eval(e, a, Int.1, k) = coeff_eval(e, a, k)
                })
                (coeff_hom_eval(coeff_tail(d), a, Int.1, k)
                    = coeff_eval(coeff_tail(d), a, k))
                coeff_hom_eval_suc(d, a, Int.1, k)
                (coeff_hom_eval(d, a, Int.1, k.suc)
                    = d(Nat.0) * Int.1.pow(k) + a * coeff_hom_eval(coeff_tail(d), a, Int.1, k))
                (Int.1.pow(k) = Int.1)
                (d(Nat.0) * Int.1 = d(Nat.0))
                (coeff_eval(d, a, k.suc) = d(Nat.0) + a * coeff_eval(coeff_tail(d), a, k))
                coeff_hom_eval(d, a, Int.1, k.suc) = coeff_eval(d, a, k.suc)
            }
            w(k.suc)
        }
        (w(k) implies w(k.suc))
    }
    w(Nat.0) and forall(k: Nat) {
        w(k) implies w(k.suc)
    }
    Nat.induction(w)
    w(n)
    (forall(d: Nat -> Int) {
        coeff_hom_eval(d, a, Int.1, n) = coeff_eval(d, a, n)
    })
    coeff_hom_eval(c, a, Int.1, n) = coeff_eval(c, a, n)
}

/// The numerator of a root divides the constant coefficient.
///
/// The first half of the rational root test. Every term above the constant one carries a factor
/// of the numerator, so a vanishing cleared value makes the constant coefficient times a power
/// of the denominator a multiple of the numerator, and coprimality strips the power.
theorem hom_root_numerator_divides(c: Nat -> Int, a: Int, b: Int, n: Nat) {
    coeff_hom_eval(c, a, b, n.suc) = Int.0 and is_coprime(a, b)
        implies a.divides(c(Nat.0))
} by {
    if coeff_hom_eval(c, a, b, n.suc) = Int.0 and is_coprime(a, b) {
        coeff_hom_eval_suc(c, a, b, n)
        (coeff_hom_eval(c, a, b, n.suc)
            = c(Nat.0) * b.pow(n) + a * coeff_hom_eval(coeff_tail(c), a, b, n))
        (Int.0 = c(Nat.0) * b.pow(n) + a * coeff_hom_eval(coeff_tail(c), a, b, n))
        (c(Nat.0) * b.pow(n) = -(a * coeff_hom_eval(coeff_tail(c), a, b, n)))
        (-(a * coeff_hom_eval(coeff_tail(c), a, b, n))
            = -coeff_hom_eval(coeff_tail(c), a, b, n) * a)
        (-coeff_hom_eval(coeff_tail(c), a, b, n) * a = c(Nat.0) * b.pow(n))
        exists(d: Int) {
            d * a = c(Nat.0) * b.pow(n)
        }
        (a.divides(c(Nat.0) * b.pow(n))
            = exists(d: Int) { d * a = c(Nat.0) * b.pow(n) })
        a.divides(c(Nat.0) * b.pow(n))
        divides_of_coprime_pow(a, c(Nat.0), b, n)
        a.divides(c(Nat.0))
    }
}

/// Adding then subtracting the same integer changes nothing.
///
/// Stated once, since the induction step below rearranges a three-term sum against its own last
/// term and neither the grouping nor the cancellation is found on its own.
theorem int_add_sub_cancel(x: Int, y: Int) {
    (x + y) - y = x
}

/// Subtracting an integer from itself gives zero.
theorem int_sub_self(x: Int) {
    x - x = Int.0
}

/// Away from the leading term, the homogeneous value is a multiple of the denominator.
///
/// Every term but the top one carries a factor of the denominator, since only the top one has
/// exponent zero on it. Proved by induction along the coefficient list: the constant term
/// carries the whole power, and the rest is the numerator times the same statement one step in.
theorem hom_eval_minus_top_divisible(c: Nat -> Int, a: Int, b: Int, n: Nat) {
    b.divides(coeff_hom_eval(c, a, b, n.suc) - c(n) * a.pow(n))
} by {
    define w(k: Nat) -> Bool {
        forall(d: Nat -> Int) {
            b.divides(coeff_hom_eval(d, a, b, k.suc) - d(k) * a.pow(k))
        }
    }
    forall(d: Nat -> Int) {
        coeff_hom_eval_suc(d, a, b, Nat.0)
        (coeff_hom_eval(d, a, b, Nat.0.suc)
            = d(Nat.0) * b.pow(Nat.0) + a * coeff_hom_eval(coeff_tail(d), a, b, Nat.0))
        coeff_hom_eval_zero(coeff_tail(d), a, b)
        (coeff_hom_eval(coeff_tail(d), a, b, Nat.0) = Int.0)
        exp_zero(b)
        (b.pow(Nat.0) = Int.1)
        (d(Nat.0) * Int.1 = d(Nat.0))
        (a * Int.0 = Int.0)
        (d(Nat.0) + Int.0 = d(Nat.0))
        (coeff_hom_eval(d, a, b, Nat.0.suc) = d(Nat.0))
        exp_zero(a)
        (a.pow(Nat.0) = Int.1)
        (d(Nat.0) * a.pow(Nat.0) = d(Nat.0))
        int_sub_self(d(Nat.0))
        (d(Nat.0) - d(Nat.0) = Int.0)
        (coeff_hom_eval(d, a, b, Nat.0.suc) - d(Nat.0) * a.pow(Nat.0) = Int.0)
        (Int.0 * b = Int.0)
        exists(e: Int) {
            e * b = coeff_hom_eval(d, a, b, Nat.0.suc) - d(Nat.0) * a.pow(Nat.0)
        }
        (b.divides(coeff_hom_eval(d, a, b, Nat.0.suc) - d(Nat.0) * a.pow(Nat.0))
            = exists(e: Int) {
                e * b = coeff_hom_eval(d, a, b, Nat.0.suc) - d(Nat.0) * a.pow(Nat.0)
            })
        b.divides(coeff_hom_eval(d, a, b, Nat.0.suc) - d(Nat.0) * a.pow(Nat.0))
    }
    w(Nat.0)
    forall(k: Nat) {
        if w(k) {
            (forall(e: Nat -> Int) {
                b.divides(coeff_hom_eval(e, a, b, k.suc) - e(k) * a.pow(k))
            })
            forall(d: Nat -> Int) {
                b.divides(coeff_hom_eval(coeff_tail(d), a, b, k.suc)
                    - coeff_tail(d)(k) * a.pow(k))
                (coeff_tail(d)(k) = d(k.suc))
                (b.divides(coeff_hom_eval(coeff_tail(d), a, b, k.suc) - d(k.suc) * a.pow(k))
                    = exists(e: Int) {
                        e * b = coeff_hom_eval(coeff_tail(d), a, b, k.suc)
                            - d(k.suc) * a.pow(k)
                    })
                let (m: Int) satisfy {
                    m * b = coeff_hom_eval(coeff_tail(d), a, b, k.suc) - d(k.suc) * a.pow(k)
                }
                coeff_hom_eval_suc(d, a, b, k.suc)
                (coeff_hom_eval(d, a, b, k.suc.suc)
                    = d(Nat.0) * b.pow(k.suc)
                        + a * coeff_hom_eval(coeff_tail(d), a, b, k.suc))
                (coeff_hom_eval(coeff_tail(d), a, b, k.suc) = m * b + d(k.suc) * a.pow(k))
                (a * (m * b + d(k.suc) * a.pow(k))
                    = a * (m * b) + a * (d(k.suc) * a.pow(k)))
                exp_one(a)
                (a.pow(Nat.1) = a)
                exp_add(a, k, Nat.1)
                (a.pow(k + Nat.1) = a.pow(k) * a.pow(Nat.1))
                (k + Nat.1 = k.suc)
                (a.pow(k.suc) = a.pow(k) * a)
                (a * (d(k.suc) * a.pow(k)) = (a * d(k.suc)) * a.pow(k))
                (a * d(k.suc) = d(k.suc) * a)
                ((d(k.suc) * a) * a.pow(k) = d(k.suc) * (a * a.pow(k)))
                (a * a.pow(k) = a.pow(k) * a)
                (a * (d(k.suc) * a.pow(k)) = d(k.suc) * a.pow(k.suc))
                (coeff_hom_eval(d, a, b, k.suc.suc)
                    = d(Nat.0) * b.pow(k.suc) + (a * (m * b) + d(k.suc) * a.pow(k.suc)))
                (d(Nat.0) * b.pow(k.suc) + (a * (m * b) + d(k.suc) * a.pow(k.suc))
                    = (d(Nat.0) * b.pow(k.suc) + a * (m * b)) + d(k.suc) * a.pow(k.suc))
                int_add_sub_cancel(d(Nat.0) * b.pow(k.suc) + a * (m * b),
                    d(k.suc) * a.pow(k.suc))
                ((d(Nat.0) * b.pow(k.suc) + a * (m * b) + d(k.suc) * a.pow(k.suc))
                    - d(k.suc) * a.pow(k.suc)
                    = d(Nat.0) * b.pow(k.suc) + a * (m * b))
                (coeff_hom_eval(d, a, b, k.suc.suc) - d(k.suc) * a.pow(k.suc)
                    = d(Nat.0) * b.pow(k.suc) + a * (m * b))
                exp_one(b)
                (b.pow(Nat.1) = b)
                exp_add(b, k, Nat.1)
                (b.pow(k + Nat.1) = b.pow(k) * b.pow(Nat.1))
                (b.pow(k.suc) = b.pow(k) * b)
                (d(Nat.0) * b.pow(k.suc) = (d(Nat.0) * b.pow(k)) * b)
                (a * (m * b) = (a * m) * b)
                ((d(Nat.0) * b.pow(k)) * b + (a * m) * b
                    = (d(Nat.0) * b.pow(k) + a * m) * b)
                ((d(Nat.0) * b.pow(k) + a * m) * b
                    = coeff_hom_eval(d, a, b, k.suc.suc) - d(k.suc) * a.pow(k.suc))
                exists(e: Int) {
                    e * b = coeff_hom_eval(d, a, b, k.suc.suc) - d(k.suc) * a.pow(k.suc)
                }
                (b.divides(coeff_hom_eval(d, a, b, k.suc.suc) - d(k.suc) * a.pow(k.suc))
                    = exists(e: Int) {
                        e * b = coeff_hom_eval(d, a, b, k.suc.suc)
                            - d(k.suc) * a.pow(k.suc)
                    })
                b.divides(coeff_hom_eval(d, a, b, k.suc.suc) - d(k.suc) * a.pow(k.suc))
            }
            w(k.suc)
        }
        (w(k) implies w(k.suc))
    }
    w(Nat.0) and forall(k: Nat) {
        w(k) implies w(k.suc)
    }
    Nat.induction(w)
    w(n)
    (forall(d: Nat -> Int) {
        b.divides(coeff_hom_eval(d, a, b, n.suc) - d(n) * a.pow(n))
    })
    b.divides(coeff_hom_eval(c, a, b, n.suc) - c(n) * a.pow(n))
}

/// The denominator of a root divides the leading coefficient.
///
/// The second half of the rational root test. Every term but the leading one carries a factor of
/// the denominator, so a vanishing cleared value makes the leading coefficient times a power of
/// the numerator a multiple of the denominator, and coprimality strips the power.
theorem hom_root_denominator_divides(c: Nat -> Int, a: Int, b: Int, n: Nat) {
    coeff_hom_eval(c, a, b, n.suc) = Int.0 and is_coprime(a, b)
        implies b.divides(c(n))
} by {
    if coeff_hom_eval(c, a, b, n.suc) = Int.0 and is_coprime(a, b) {
        hom_eval_minus_top_divisible(c, a, b, n)
        b.divides(coeff_hom_eval(c, a, b, n.suc) - c(n) * a.pow(n))
        (coeff_hom_eval(c, a, b, n.suc) - c(n) * a.pow(n) = Int.0 - c(n) * a.pow(n))
        (Int.0 - c(n) * a.pow(n) = -(c(n) * a.pow(n)))
        b.divides(-(c(n) * a.pow(n)))
        (b.divides(-(c(n) * a.pow(n)))
            = exists(e: Int) { e * b = -(c(n) * a.pow(n)) })
        let (m: Int) satisfy {
            m * b = -(c(n) * a.pow(n))
        }
        (-m * b = c(n) * a.pow(n))
        exists(e: Int) {
            e * b = c(n) * a.pow(n)
        }
        (b.divides(c(n) * a.pow(n)) = exists(e: Int) { e * b = c(n) * a.pow(n) })
        b.divides(c(n) * a.pow(n))
        is_coprime_symm(a, b)
        is_coprime(b, a)
        divides_of_coprime_pow(b, c(n), a, n)
        b.divides(c(n))
    }
}
