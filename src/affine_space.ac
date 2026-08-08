from algebra.add_comm_group import AddCommGroup
from algebra.zero import Zero
from pair import Pair, pair_ext
from algebra.product_algebra import pair_add_first, pair_add_second, pair_zero_first, pair_zero_second
from data.basic.pair_algebraic_instances import pair_typeclass_add_eq_pair_add,
    pair_typeclass_zero_eq_pair_zero

/// True if zero translates every point to itself.
define affine_zero_vadd_constraint[V: AddCommGroup, P](vadd: V -> P -> P) -> Bool {
    forall(p: P) {
        vadd(V.0, p) = p
    }
}

/// True if translation by a sum equals iterated translation.
define affine_add_vadd_constraint[V: AddCommGroup, P](vadd: V -> P -> P) -> Bool {
    forall(v: V, w: V, p: P) {
        vadd(v + w, p) = vadd(v, vadd(w, p))
    }
}

/// True if subtracting then translating recovers the first point.
define affine_vsub_vadd_constraint[V: AddCommGroup, P](
    vadd: V -> P -> P, vsub: P -> P -> V) -> Bool {
    forall(p: P, q: P) {
        vadd(vsub(p, q), q) = p
    }
}

/// True if translating then subtracting recovers the translation vector.
define affine_vadd_vsub_constraint[V: AddCommGroup, P](
    vadd: V -> P -> P, vsub: P -> P -> V) -> Bool {
    forall(v: V, p: P) {
        vsub(vadd(v, p), p) = v
    }
}

/// True if vadd and vsub make P an affine space (a torsor) over V.
define is_affine_space[V: AddCommGroup, P](
    vadd: V -> P -> P, vsub: P -> P -> V) -> Bool {
    affine_zero_vadd_constraint(vadd)
    and affine_add_vadd_constraint(vadd)
    and affine_vsub_vadd_constraint(vadd, vsub)
    and affine_vadd_vsub_constraint(vadd, vsub)
}

/// An affine space over V: a type P together with a free and transitive action
/// of the additive commutative group V on P.
structure AffineSpace[V: AddCommGroup, P] {
    /// Translation of a point by a vector.
    vadd: V -> P -> P
    /// Difference of two points, returning the vector from the second to the first.
    vsub: P -> P -> V
} constraint {
    is_affine_space(vadd, vsub)
}

/// Translation in an affine space constructed from specified operations.
theorem affine_space_new_vadd_apply[V: AddCommGroup, P](
    vadd: V -> P -> P, vsub: P -> P -> V, a: AffineSpace[V, P], v: V, p: P
) {
    AffineSpace.new(vadd, vsub) = Option.some(a) implies a.vadd(v, p) = vadd(v, p)
} by {
    if AffineSpace.new(vadd, vsub) = Option.some(a) {
        a.vadd(v, p) = vadd(v, p)
    }
}

/// Subtraction in an affine space constructed from specified operations.
theorem affine_space_new_vsub_apply[V: AddCommGroup, P](
    vadd: V -> P -> P, vsub: P -> P -> V, a: AffineSpace[V, P], p: P, q: P
) {
    AffineSpace.new(vadd, vsub) = Option.some(a) implies a.vsub(p, q) = vsub(p, q)
} by {
    if AffineSpace.new(vadd, vsub) = Option.some(a) {
        a.vsub(p, q) = vsub(p, q)
    }
}

/// Translation by zero is the identity.
theorem affine_zero_vadd[V: AddCommGroup, P](a: AffineSpace[V, P], p: P) {
    a.vadd(V.0, p) = p
} by {
    is_affine_space(a.vadd, a.vsub) =
        (affine_zero_vadd_constraint(a.vadd)
         and affine_add_vadd_constraint(a.vadd)
         and affine_vsub_vadd_constraint(a.vadd, a.vsub)
         and affine_vadd_vsub_constraint(a.vadd, a.vsub))
}

/// Translation by a sum equals iterated translation.
theorem affine_add_vadd[V: AddCommGroup, P](
    a: AffineSpace[V, P], v: V, w: V, p: P) {
    a.vadd(v + w, p) = a.vadd(v, a.vadd(w, p))
} by {
    is_affine_space(a.vadd, a.vsub) =
        (affine_zero_vadd_constraint(a.vadd)
         and affine_add_vadd_constraint(a.vadd)
         and affine_vsub_vadd_constraint(a.vadd, a.vsub)
         and affine_vadd_vsub_constraint(a.vadd, a.vsub))
    affine_add_vadd_constraint(a.vadd) = forall(x: V, y: V, q: P) {
        a.vadd(x + y, q) = a.vadd(x, a.vadd(y, q))
    }
}

/// Subtracting and then translating recovers the first point.
theorem affine_vsub_vadd[V: AddCommGroup, P](
    a: AffineSpace[V, P], p: P, q: P) {
    a.vadd(a.vsub(p, q), q) = p
} by {
    is_affine_space(a.vadd, a.vsub) =
        (affine_zero_vadd_constraint(a.vadd)
         and affine_add_vadd_constraint(a.vadd)
         and affine_vsub_vadd_constraint(a.vadd, a.vsub)
         and affine_vadd_vsub_constraint(a.vadd, a.vsub))
    affine_vsub_vadd_constraint(a.vadd, a.vsub) = forall(x: P, y: P) {
        a.vadd(a.vsub(x, y), y) = x
    }
}

/// Translating then subtracting recovers the translation vector.
theorem affine_vadd_vsub[V: AddCommGroup, P](
    a: AffineSpace[V, P], v: V, p: P) {
    a.vsub(a.vadd(v, p), p) = v
} by {
    affine_vadd_vsub_constraint(a.vadd, a.vsub)
    affine_vadd_vsub_constraint(a.vadd, a.vsub) = forall(x: V, q: P) {
        a.vsub(a.vadd(x, q), q) = x
    }
}

/// Subtracting a point from itself yields the zero vector.
theorem affine_vsub_self[V: AddCommGroup, P](a: AffineSpace[V, P], p: P) {
    a.vsub(p, p) = V.0
} by {
    affine_zero_vadd(a, p)
    affine_vadd_vsub(a, V.0, p)
}

/// Translation is injective in the point argument.
theorem affine_vadd_left_cancel[V: AddCommGroup, P](
    a: AffineSpace[V, P], v: V, p: P, q: P) {
    a.vadd(v, p) = a.vadd(v, q) implies p = q
} by {
    if a.vadd(v, p) = a.vadd(v, q) {
        let nv: V = -v
        nv + v = V.0
        affine_add_vadd(a, nv, v, p)
        affine_add_vadd(a, nv, v, q)
        affine_zero_vadd(a, p)
        affine_zero_vadd(a, q)
        p = q
    }
}

/// Translation is injective in the vector argument.
theorem affine_vadd_right_cancel[V: AddCommGroup, P](
    a: AffineSpace[V, P], v: V, w: V, p: P) {
    a.vadd(v, p) = a.vadd(w, p) implies v = w
} by {
    if a.vadd(v, p) = a.vadd(w, p) {
        affine_vadd_vsub(a, v, p)
        affine_vadd_vsub(a, w, p)
        a.vsub(a.vadd(w, p), p) = w
    }
}

/// Two points are equal exactly when their difference is zero.
theorem affine_vsub_eq_zero[V: AddCommGroup, P](
    a: AffineSpace[V, P], p: P, q: P) {
    a.vsub(p, q) = V.0 implies p = q
} by {
    if a.vsub(p, q) = V.0 {
        affine_vsub_vadd(a, p, q)
        affine_zero_vadd(a, q)
        a.vadd(V.0, q) = q
    }
}

/// Componentwise translation on a product of affine spaces.
define affine_space_product_vadd[V: AddCommGroup, W: AddCommGroup, P, Q](
    a: AffineSpace[V, P], b: AffineSpace[W, Q], v: Pair[V, W], p: Pair[P, Q]
) -> Pair[P, Q] {
    Pair.new(a.vadd(v.first, p.first), b.vadd(v.second, p.second))
}

/// Componentwise subtraction on a product of affine spaces.
define affine_space_product_vsub[V: AddCommGroup, W: AddCommGroup, P, Q](
    a: AffineSpace[V, P], b: AffineSpace[W, Q], p: Pair[P, Q], q: Pair[P, Q]
) -> Pair[V, W] {
    Pair.new(a.vsub(p.first, q.first), b.vsub(p.second, q.second))
}

/// The first coordinate of componentwise product translation.
theorem affine_space_product_vadd_first[V: AddCommGroup, W: AddCommGroup, P, Q](
    a: AffineSpace[V, P], b: AffineSpace[W, Q], v: Pair[V, W], p: Pair[P, Q]
) {
    affine_space_product_vadd(a, b, v, p).first = a.vadd(v.first, p.first)
}

/// The second coordinate of componentwise product translation.
theorem affine_space_product_vadd_second[V: AddCommGroup, W: AddCommGroup, P, Q](
    a: AffineSpace[V, P], b: AffineSpace[W, Q], v: Pair[V, W], p: Pair[P, Q]
) {
    affine_space_product_vadd(a, b, v, p).second = b.vadd(v.second, p.second)
}

/// The first coordinate of componentwise product subtraction.
theorem affine_space_product_vsub_first[V: AddCommGroup, W: AddCommGroup, P, Q](
    a: AffineSpace[V, P], b: AffineSpace[W, Q], p: Pair[P, Q], q: Pair[P, Q]
) {
    affine_space_product_vsub(a, b, p, q).first = a.vsub(p.first, q.first)
}

/// The second coordinate of componentwise product subtraction.
theorem affine_space_product_vsub_second[V: AddCommGroup, W: AddCommGroup, P, Q](
    a: AffineSpace[V, P], b: AffineSpace[W, Q], p: Pair[P, Q], q: Pair[P, Q]
) {
    affine_space_product_vsub(a, b, p, q).second = b.vsub(p.second, q.second)
}

/// Componentwise translation and subtraction form an affine space on a product.
theorem affine_space_product_is_affine_space[V: AddCommGroup, W: AddCommGroup, P, Q](
    a: AffineSpace[V, P], b: AffineSpace[W, Q]
) {
    is_affine_space(affine_space_product_vadd(a, b), affine_space_product_vsub(a, b))
} by {
    forall(p: Pair[P, Q]) {
        pair_typeclass_zero_eq_pair_zero[V, W]
        pair_zero_first[V, W]
        pair_zero_second[V, W]
        affine_zero_vadd(a, p.first)
        affine_zero_vadd(b, p.second)
        let translated = affine_space_product_vadd(a, b, Zero.0[Pair[V, W]], p)
        affine_space_product_vadd_first(a, b, Zero.0[Pair[V, W]], p)
        affine_space_product_vadd_second(a, b, Zero.0[Pair[V, W]], p)
        translated.first = p.first
        translated.second = p.second
        pair_ext(translated, p)
        affine_space_product_vadd(a, b, Zero.0[Pair[V, W]], p) = p
    }
    affine_zero_vadd_constraint(affine_space_product_vadd(a, b))

    forall(v: Pair[V, W], w: Pair[V, W], p: Pair[P, Q]) {
        pair_typeclass_add_eq_pair_add(v, w)
        pair_add_first(v, w)
        pair_add_second(v, w)
        (v + w).first = v.first + w.first
        (v + w).second = v.second + w.second
        affine_add_vadd(a, v.first, w.first, p.first)
        affine_add_vadd(b, v.second, w.second, p.second)
        let lhs = affine_space_product_vadd(a, b, v + w, p)
        let inner = affine_space_product_vadd(a, b, w, p)
        let rhs = affine_space_product_vadd(a, b, v, inner)
        affine_space_product_vadd_first(a, b, v + w, p)
        affine_space_product_vadd_first(a, b, w, p)
        affine_space_product_vadd_first(a, b, v, inner)
        lhs.first = a.vadd(v.first + w.first, p.first)
        rhs.first = a.vadd(v.first, a.vadd(w.first, p.first))
        lhs.first = rhs.first
        affine_space_product_vadd_second(a, b, v + w, p)
        affine_space_product_vadd_second(a, b, w, p)
        affine_space_product_vadd_second(a, b, v, inner)
        lhs.second = b.vadd(v.second + w.second, p.second)
        rhs.second = b.vadd(v.second, b.vadd(w.second, p.second))
        lhs.second = rhs.second
        pair_ext(lhs, rhs)
        affine_space_product_vadd(a, b, v + w, p) =
            affine_space_product_vadd(a, b, v, affine_space_product_vadd(a, b, w, p))
    }
    affine_add_vadd_constraint(affine_space_product_vadd(a, b))

    forall(p: Pair[P, Q], q: Pair[P, Q]) {
        affine_vsub_vadd(a, p.first, q.first)
        affine_vsub_vadd(b, p.second, q.second)
        affine_space_product_vsub_first(a, b, p, q)
        affine_space_product_vsub_second(a, b, p, q)
        let recovered = affine_space_product_vadd(a, b, affine_space_product_vsub(a, b, p, q), q)
        affine_space_product_vadd_first(a, b, affine_space_product_vsub(a, b, p, q), q)
        affine_space_product_vadd_second(a, b, affine_space_product_vsub(a, b, p, q), q)
        recovered.first = a.vadd(a.vsub(p.first, q.first), q.first)
        recovered.first = p.first
        recovered.second = b.vadd(b.vsub(p.second, q.second), q.second)
        recovered.second = p.second
        pair_ext(recovered, p)
        affine_space_product_vadd(a, b, affine_space_product_vsub(a, b, p, q), q) = p
    }
    affine_vsub_vadd_constraint(
        affine_space_product_vadd(a, b), affine_space_product_vsub(a, b))

    forall(v: Pair[V, W], p: Pair[P, Q]) {
        affine_vadd_vsub(a, v.first, p.first)
        affine_vadd_vsub(b, v.second, p.second)
        affine_space_product_vadd_first(a, b, v, p)
        affine_space_product_vadd_second(a, b, v, p)
        let recovered = affine_space_product_vsub(
            a, b, affine_space_product_vadd(a, b, v, p), p)
        affine_space_product_vsub_first(a, b, affine_space_product_vadd(a, b, v, p), p)
        affine_space_product_vsub_second(a, b, affine_space_product_vadd(a, b, v, p), p)
        recovered.first = a.vsub(a.vadd(v.first, p.first), p.first)
        recovered.first = v.first
        recovered.second = b.vsub(b.vadd(v.second, p.second), p.second)
        recovered.second = v.second
        pair_ext(recovered, v)
        affine_space_product_vsub(a, b, affine_space_product_vadd(a, b, v, p), p) = v
    }
    affine_vadd_vsub_constraint(
        affine_space_product_vadd(a, b), affine_space_product_vsub(a, b))
}

/// The product of two affine spaces, with operations defined componentwise.
let affine_space_product[V: AddCommGroup, W: AddCommGroup, P, Q](a: AffineSpace[V, P], b: AffineSpace[W, Q]) -> result: AffineSpace[Pair[V, W], Pair[P, Q]] satisfy {
    AffineSpace.new(affine_space_product_vadd(a, b), affine_space_product_vsub(a, b)) =
        Option.some(result)
}

/// Translation in a product affine space is componentwise.
theorem affine_space_product_vadd_apply[V: AddCommGroup, W: AddCommGroup, P, Q](
    a: AffineSpace[V, P], b: AffineSpace[W, Q], v: Pair[V, W], p: Pair[P, Q]
) {
    affine_space_product(a, b).vadd(v, p) =
        Pair.new(a.vadd(v.first, p.first), b.vadd(v.second, p.second))
} by {
    AffineSpace.new(affine_space_product_vadd(a, b), affine_space_product_vsub(a, b)) =
        Option.some(affine_space_product(a, b))
    affine_space_new_vadd_apply(
        affine_space_product_vadd(a, b), affine_space_product_vsub(a, b),
        affine_space_product(a, b), v, p)
}

/// Subtraction in a product affine space is componentwise.
theorem affine_space_product_vsub_apply[V: AddCommGroup, W: AddCommGroup, P, Q](
    a: AffineSpace[V, P], b: AffineSpace[W, Q], p: Pair[P, Q], q: Pair[P, Q]
) {
    affine_space_product(a, b).vsub(p, q) =
        Pair.new(a.vsub(p.first, q.first), b.vsub(p.second, q.second))
} by {
    AffineSpace.new(affine_space_product_vadd(a, b), affine_space_product_vsub(a, b)) =
        Option.some(affine_space_product(a, b))
    affine_space_new_vsub_apply(
        affine_space_product_vadd(a, b), affine_space_product_vsub(a, b),
        affine_space_product(a, b), p, q)
}

/// Translation of a point by a vector when the point type is the additive commutative group itself.
let add_comm_group_vadd[V: AddCommGroup]: V -> V -> V = function(v: V, p: V) {
    v + p
}

/// Difference of two points when the point type is the additive commutative group itself.
let add_comm_group_vsub[V: AddCommGroup]: V -> V -> V = function(p: V, q: V) {
    p - q
}

/// The canonical translation and subtraction make V an affine space over itself.
theorem add_comm_group_is_affine_space[V: AddCommGroup] {
    is_affine_space(add_comm_group_vadd[V], add_comm_group_vsub[V])
} by {
    forall(p: V) {
        add_comm_group_vadd[V](V.0, p) = p
    }
    affine_zero_vadd_constraint(add_comm_group_vadd[V])

    forall(v: V, w: V, p: V) {
        add_comm_group_vadd[V](v, add_comm_group_vadd[V](w, p)) = v + (w + p)
        add_comm_group_vadd[V](v + w, p) = add_comm_group_vadd[V](v, add_comm_group_vadd[V](w, p))
    }
    affine_add_vadd_constraint(add_comm_group_vadd[V])

    forall(p: V, q: V) {
        add_comm_group_vadd[V](add_comm_group_vsub[V](p, q), q) = (p + -q) + q
        (p + -q) + q = p + (-q + q)
        -q + q = V.0
        add_comm_group_vadd[V](add_comm_group_vsub[V](p, q), q) = p
    }
    affine_vsub_vadd_constraint(add_comm_group_vadd[V], add_comm_group_vsub[V])

    forall(v: V, p: V) {
        (v + p) - p = (v + p) + -p
        (v + p) + -p = v + (p + -p)
        p + -p = V.0
        add_comm_group_vsub[V](add_comm_group_vadd[V](v, p), p) = v
    }
    affine_vadd_vsub_constraint(add_comm_group_vadd[V], add_comm_group_vsub[V])

}

/// An additive commutative group viewed as an affine space over itself.
let add_comm_group_as_affine_space[V: AddCommGroup]: AffineSpace[V, V] satisfy {
    AffineSpace.new(add_comm_group_vadd[V], add_comm_group_vsub[V]) =
        Option.some(add_comm_group_as_affine_space)
}

/// The translation on the canonical self-affine-space is group addition.
theorem add_comm_group_as_affine_space_vadd[V: AddCommGroup](v: V, p: V) {
    add_comm_group_as_affine_space[V].vadd(v, p) = v + p
} by {
    add_comm_group_as_affine_space[V].vadd = add_comm_group_vadd[V]
}

/// The subtraction on the canonical self-affine-space is group subtraction.
theorem add_comm_group_as_affine_space_vsub[V: AddCommGroup](p: V, q: V) {
    add_comm_group_as_affine_space[V].vsub(p, q) = p - q
} by {
    add_comm_group_as_affine_space[V].vsub = add_comm_group_vsub[V]
}
