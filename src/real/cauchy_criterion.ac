/// The Cauchy criterion for real-valued sequences.
from nat import Nat
from data.basic.logic import eq_true_intro, eq_true_elim
from real.real_base import Real
from real.real_seq import cauchy_bound, converges, converges_to, converges_to_imp_converges, converges_imp_converges_to, limit

/// The Cauchy tail condition at a fixed tolerance.
define is_cauchy_seq_at(a: Nat -> Real, eps: Real) -> Bool {
    eps.is_positive implies exists(n: Nat) {
        forall(i: Nat, j: Nat) {
            n <= i and n <= j implies a(i).is_close(a(j), eps)
        }
    }
}

/// Introduce the pointwise Cauchy predicate from its expanded form.
theorem is_cauchy_seq_at_intro(a: Nat -> Real, eps: Real) {
    (eps.is_positive implies exists(n: Nat) {
        forall(i: Nat, j: Nat) {
            n <= i and n <= j implies a(i).is_close(a(j), eps)
        }
    }) implies is_cauchy_seq_at(a, eps)
} by {
    if eps.is_positive implies exists(n: Nat) {
        forall(i: Nat, j: Nat) {
            n <= i and n <= j implies a(i).is_close(a(j), eps)
        }
    } {
        is_cauchy_seq_at(a, eps) = (eps.is_positive implies exists(n: Nat) {
            forall(i: Nat, j: Nat) {
                n <= i and n <= j implies a(i).is_close(a(j), eps)
            }
        })
        eq_true_intro(eps.is_positive implies exists(n: Nat) {
            forall(i: Nat, j: Nat) {
                n <= i and n <= j implies a(i).is_close(a(j), eps)
            }
        })
        (eps.is_positive implies exists(n: Nat) {
            forall(i: Nat, j: Nat) {
                n <= i and n <= j implies a(i).is_close(a(j), eps)
            }
        }) = true
        is_cauchy_seq_at(a, eps) = true
        eq_true_elim(is_cauchy_seq_at(a, eps))
        is_cauchy_seq_at(a, eps)
    }
}

/// True if a is a Cauchy sequence: for every positive epsilon, terms past
/// some index are within epsilon of one another.
define is_cauchy_seq(a: Nat -> Real) -> Bool {
    forall(eps: Real) {
        is_cauchy_seq_at(a, eps)
    }
}

/// Introduce the Cauchy sequence predicate from its pointwise form.
theorem is_cauchy_seq_intro(a: Nat -> Real) {
    (forall(eps: Real) { is_cauchy_seq_at(a, eps) }) implies is_cauchy_seq(a)
} by {
    if forall(eps: Real) { is_cauchy_seq_at(a, eps) } {
        is_cauchy_seq(a) = forall(eps: Real) {
            is_cauchy_seq_at(a, eps)
        }
    }
}

/// A Cauchy tail bound controls every pair of indices beyond the bound.
theorem cauchy_bound_all_indices(a: Nat -> Real, n: Nat, eps: Real) {
    cauchy_bound(a, n, eps) implies forall(i: Nat, j: Nat) {
        n <= i and n <= j implies a(i).is_close(a(j), eps)
    }
} by {
    if cauchy_bound(a, n, eps) {
        cauchy_bound(a, n, eps) = forall(i: Nat, j: Nat) {
            n <= i and n <= j implies a(i).is_close(a(j), eps)
        }
    }
}

/// A Cauchy tail bound controls a chosen pair of indices beyond the bound.
theorem cauchy_bound_indices(a: Nat -> Real, n: Nat, eps: Real, i: Nat, j: Nat) {
    cauchy_bound(a, n, eps) and n <= i and n <= j implies a(i).is_close(a(j), eps)
} by {
    if cauchy_bound(a, n, eps) and n <= i and n <= j {
        cauchy_bound_all_indices(a, n, eps)
        n <= i and n <= j implies a(i).is_close(a(j), eps)
        a(i).is_close(a(j), eps)
    }
}

/// Every Cauchy sequence satisfies the existing `converges` predicate.
theorem cauchy_imp_converges(a: Nat -> Real) {
    is_cauchy_seq(a) implies converges(a)
} by {
    if is_cauchy_seq(a) {
        is_cauchy_seq(a) = forall(eps: Real) {
            is_cauchy_seq_at(a, eps)
        }
        forall(eps: Real) {
            if eps.is_positive {
                is_cauchy_seq_at(a, eps)
                is_cauchy_seq_at(a, eps) = (eps.is_positive implies exists(n: Nat) {
                    forall(i: Nat, j: Nat) {
                        n <= i and n <= j implies a(i).is_close(a(j), eps)
                    }
                })
                eps.is_positive implies exists(n: Nat) {
                    forall(i: Nat, j: Nat) {
                        n <= i and n <= j implies a(i).is_close(a(j), eps)
                    }
                }
                let n: Nat satisfy {
                    forall(i: Nat, j: Nat) {
                        n <= i and n <= j implies a(i).is_close(a(j), eps)
                    }
                }
                cauchy_bound(a, n, eps) = forall(i: Nat, j: Nat) {
                    n <= i and n <= j implies a(i).is_close(a(j), eps)
                }
                cauchy_bound(a, n, eps)
            }
        }
    }
}

/// Helper: any positive epsilon admits a Cauchy tail bound.
theorem converges_imp_cauchy_bound(a: Nat -> Real, eps: Real) {
    converges(a) and eps.is_positive implies exists(n: Nat) {
        cauchy_bound(a, n, eps)
    }
} by {
    if converges(a) and eps.is_positive {
        converges(a) = forall(e: Real) {
            e.is_positive implies exists(n: Nat) {
                cauchy_bound(a, n, e)
            }
        }
        eps.is_positive implies exists(n: Nat) {
            cauchy_bound(a, n, eps)
        }
    }
}

/// Helper: any positive epsilon admits an explicit Cauchy tail.
theorem converges_imp_explicit_tail(a: Nat -> Real, eps: Real) {
    converges(a) and eps.is_positive implies exists(n: Nat) {
        forall(i: Nat, j: Nat) {
            n <= i and n <= j implies a(i).is_close(a(j), eps)
        }
    }
} by {
    if converges(a) and eps.is_positive {
        converges_imp_cauchy_bound(a, eps)
        let n: Nat satisfy {
            cauchy_bound(a, n, eps)
        }
        cauchy_bound_all_indices(a, n, eps)
        exists(m: Nat) {
            forall(i: Nat, j: Nat) {
                m <= i and m <= j implies a(i).is_close(a(j), eps)
            }
        }
    }
}

/// Every sequence satisfying `converges` is Cauchy in the explicit sense.
theorem converges_imp_cauchy_seq(a: Nat -> Real) {
    converges(a) implies is_cauchy_seq(a)
} by {
    if converges(a) {
        forall(eps: Real) {
            if eps.is_positive {
                converges_imp_explicit_tail(a, eps)
                exists(n: Nat) {
                    forall(i: Nat, j: Nat) {
                        n <= i and n <= j implies a(i).is_close(a(j), eps)
                    }
                }
            }
        }
        forall(eps: Real) {
            is_cauchy_seq_at(a, eps) = (eps.is_positive implies exists(n: Nat) {
                forall(i: Nat, j: Nat) {
                    n <= i and n <= j implies a(i).is_close(a(j), eps)
                }
            })
            eps.is_positive implies exists(n: Nat) {
                forall(i: Nat, j: Nat) {
                    n <= i and n <= j implies a(i).is_close(a(j), eps)
                }
            }
            is_cauchy_seq_at_intro(a, eps)
            is_cauchy_seq_at(a, eps)
        }
        is_cauchy_seq_intro(a)
        is_cauchy_seq(a)
    }
}

/// Every sequence convergent to a limit is Cauchy.
theorem converges_to_imp_cauchy(a: Nat -> Real, x: Real) {
    converges_to(a, x) implies is_cauchy_seq(a)
} by {
    if converges_to(a, x) {
        converges_to_imp_converges(a, x)
        converges_imp_cauchy_seq(a)
    }
}

/// A Cauchy real sequence converges to its limit.
theorem cauchy_imp_converges_to_limit(a: Nat -> Real) {
    is_cauchy_seq(a) implies converges_to(a, limit(a))
} by {
    if is_cauchy_seq(a) {
        cauchy_imp_converges(a)
        converges_imp_converges_to(a)
    }
}

/// A Cauchy real sequence has some real limit.
theorem cauchy_imp_exists_limit(a: Nat -> Real) {
    is_cauchy_seq(a) implies exists(x: Real) {
        converges_to(a, x)
    }
} by {
    if is_cauchy_seq(a) {
        cauchy_imp_converges_to_limit(a)
    }
}

/// Cauchy criterion (one direction): a Cauchy sequence has a real limit.
theorem cauchy_iff_exists_limit_forward(a: Nat -> Real) {
    is_cauchy_seq(a) implies exists(x: Real) {
        converges_to(a, x)
    }
} by {
    cauchy_imp_exists_limit(a)
}

/// Cauchy criterion (other direction): a sequence with a real limit is Cauchy.
theorem cauchy_iff_exists_limit_backward(a: Nat -> Real) {
    (exists(x: Real) { converges_to(a, x) }) implies is_cauchy_seq(a)
} by {
    if exists(x: Real) { converges_to(a, x) } {
        let x: Real satisfy {
            converges_to(a, x)
        }
        converges_to_imp_cauchy(a, x)
    }
}
