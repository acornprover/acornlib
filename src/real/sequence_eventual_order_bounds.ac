/// Eventual lower and upper real-sequence bounds under tail/subsequence transport.
///
/// This module packages order-bound consequences of the accepted Nat-eventual
/// and sequence tail/subsequence transport APIs.  It intentionally introduces no
/// new generic eventual API, no growth-rate notation, and no extra barrel edit.

from data.basic.functions import compose
from nat import Nat
from data.basic.set import Set
from real.real_field import Real
from real.real_seq import converges, converges_to, converges_to_imp_converges,
    limit, eventual_lb, eventual_ub, lb_lte_limit, ub_imp_limit_lte
from real.topology_rays import closed_upper_ray, closed_lower_ray,
    closed_upper_ray_contains_eq, closed_lower_ray_contains_eq
from real.sequence_set_membership import seq_eventually_in_real_set
from real.sequence_eventual_tail_transport import seq_eventually_in_real_set_compose_tends_to_infinity,
    seq_eventually_in_real_set_shift_add,
    seq_eventually_in_real_set_subsequence,
    seq_eventually_in_real_set_tail_subsequence
from real.limits import tends_to_infinity, is_subsequence_index, subsequence,
    converges_compose_tends_to_infinity, converges_compose_add,
    converges_subsequence, converges_subsequence_add

/// Eventual lower bounds are exactly eventual membership in the closed upper ray.
theorem eventual_lb_iff_seq_eventually_in_closed_upper_ray(a: Nat -> Real, lb: Real) {
    eventual_lb(a, lb) = seq_eventually_in_real_set(closed_upper_ray(lb), a)
} by {
    if eventual_lb(a, lb) {
        let n0: Nat satisfy {
            forall(n: Nat) {
                n0 <= n implies lb <= a(n)
            }
        }
        forall(n: Nat) {
            if n0 <= n {
                lb <= a(n)
                closed_upper_ray_contains_eq(lb, a(n))
                closed_upper_ray(lb).contains(a(n))
            }
        }
        exists(m: Nat) {
            forall(n: Nat) {
                m <= n implies closed_upper_ray(lb).contains(a(n))
            }
        }
        seq_eventually_in_real_set(closed_upper_ray(lb), a)
    }
    if seq_eventually_in_real_set(closed_upper_ray(lb), a) {
        let n0: Nat satisfy {
            forall(n: Nat) {
                n0 <= n implies closed_upper_ray(lb).contains(a(n))
            }
        }
        forall(n: Nat) {
            if n0 <= n {
                closed_upper_ray(lb).contains(a(n))
                closed_upper_ray_contains_eq(lb, a(n))
                lb <= a(n)
            }
        }
        exists(m: Nat) {
            forall(n: Nat) {
                m <= n implies lb <= a(n)
            }
        }
        eventual_lb(a, lb)
    }
}

/// Eventual upper bounds are exactly eventual membership in the closed lower ray.
theorem eventual_ub_iff_seq_eventually_in_closed_lower_ray(a: Nat -> Real, ub: Real) {
    eventual_ub(a, ub) = seq_eventually_in_real_set(closed_lower_ray(ub), a)
} by {
    if eventual_ub(a, ub) {
        let n0: Nat satisfy {
            forall(n: Nat) {
                n0 <= n implies a(n) <= ub
            }
        }
        forall(n: Nat) {
            if n0 <= n {
                a(n) <= ub
                closed_lower_ray_contains_eq(ub, a(n))
                closed_lower_ray(ub).contains(a(n))
            }
        }
        exists(m: Nat) {
            forall(n: Nat) {
                m <= n implies closed_lower_ray(ub).contains(a(n))
            }
        }
        seq_eventually_in_real_set(closed_lower_ray(ub), a)
    }
    if seq_eventually_in_real_set(closed_lower_ray(ub), a) {
        let n0: Nat satisfy {
            forall(n: Nat) {
                n0 <= n implies closed_lower_ray(ub).contains(a(n))
            }
        }
        forall(n: Nat) {
            if n0 <= n {
                closed_lower_ray(ub).contains(a(n))
                closed_lower_ray_contains_eq(ub, a(n))
                a(n) <= ub
            }
        }
        exists(m: Nat) {
            forall(n: Nat) {
                m <= n implies a(n) <= ub
            }
        }
        eventual_ub(a, ub)
    }
}

/// Eventual lower bounds are preserved by reindexing along maps tending to infinity.
theorem eventual_lb_compose_tends_to_infinity(a: Nat -> Real, f: Nat -> Nat, lb: Real) {
    eventual_lb(a, lb) and tends_to_infinity(f) implies eventual_lb(compose(a, f), lb)
} by {
    if eventual_lb(a, lb) and tends_to_infinity(f) {
        eventual_lb_iff_seq_eventually_in_closed_upper_ray(a, lb)
        seq_eventually_in_real_set(closed_upper_ray(lb), a)
        seq_eventually_in_real_set_compose_tends_to_infinity(closed_upper_ray(lb), a, f)
        seq_eventually_in_real_set(closed_upper_ray(lb), compose(a, f))
        eventual_lb_iff_seq_eventually_in_closed_upper_ray(compose(a, f), lb)
        eventual_lb(compose(a, f), lb)
    }
}

/// Eventual upper bounds are preserved by reindexing along maps tending to infinity.
theorem eventual_ub_compose_tends_to_infinity(a: Nat -> Real, f: Nat -> Nat, ub: Real) {
    eventual_ub(a, ub) and tends_to_infinity(f) implies eventual_ub(compose(a, f), ub)
} by {
    if eventual_ub(a, ub) and tends_to_infinity(f) {
        eventual_ub_iff_seq_eventually_in_closed_lower_ray(a, ub)
        seq_eventually_in_real_set(closed_lower_ray(ub), a)
        seq_eventually_in_real_set_compose_tends_to_infinity(closed_lower_ray(ub), a, f)
        seq_eventually_in_real_set(closed_lower_ray(ub), compose(a, f))
        eventual_ub_iff_seq_eventually_in_closed_lower_ray(compose(a, f), ub)
        eventual_ub(compose(a, f), ub)
    }
}

/// Eventual lower bounds are preserved by discarding a finite prefix.
theorem eventual_lb_shift_add(a: Nat -> Real, k: Nat, lb: Real) {
    eventual_lb(a, lb) implies eventual_lb(compose(a, k.add), lb)
} by {
    if eventual_lb(a, lb) {
        eventual_lb_iff_seq_eventually_in_closed_upper_ray(a, lb)
        seq_eventually_in_real_set(closed_upper_ray(lb), a)
        seq_eventually_in_real_set_shift_add(closed_upper_ray(lb), a, k)
        seq_eventually_in_real_set(closed_upper_ray(lb), compose(a, k.add))
        eventual_lb_iff_seq_eventually_in_closed_upper_ray(compose(a, k.add), lb)
        eventual_lb(compose(a, k.add), lb)
    }
}

/// Eventual upper bounds are preserved by discarding a finite prefix.
theorem eventual_ub_shift_add(a: Nat -> Real, k: Nat, ub: Real) {
    eventual_ub(a, ub) implies eventual_ub(compose(a, k.add), ub)
} by {
    if eventual_ub(a, ub) {
        eventual_ub_iff_seq_eventually_in_closed_lower_ray(a, ub)
        seq_eventually_in_real_set(closed_lower_ray(ub), a)
        seq_eventually_in_real_set_shift_add(closed_lower_ray(ub), a, k)
        seq_eventually_in_real_set(closed_lower_ray(ub), compose(a, k.add))
        eventual_ub_iff_seq_eventually_in_closed_lower_ray(compose(a, k.add), ub)
        eventual_ub(compose(a, k.add), ub)
    }
}

/// Eventual lower bounds are preserved by selected library subsequences.
theorem eventual_lb_subsequence(a: Nat -> Real, f: Nat -> Nat, lb: Real) {
    eventual_lb(a, lb) and is_subsequence_index(f) implies eventual_lb(subsequence(a, f), lb)
} by {
    if eventual_lb(a, lb) and is_subsequence_index(f) {
        eventual_lb_iff_seq_eventually_in_closed_upper_ray(a, lb)
        seq_eventually_in_real_set(closed_upper_ray(lb), a)
        seq_eventually_in_real_set_subsequence(closed_upper_ray(lb), a, f)
        seq_eventually_in_real_set(closed_upper_ray(lb), subsequence(a, f))
        eventual_lb_iff_seq_eventually_in_closed_upper_ray(subsequence(a, f), lb)
        eventual_lb(subsequence(a, f), lb)
    }
}

/// Eventual upper bounds are preserved by selected library subsequences.
theorem eventual_ub_subsequence(a: Nat -> Real, f: Nat -> Nat, ub: Real) {
    eventual_ub(a, ub) and is_subsequence_index(f) implies eventual_ub(subsequence(a, f), ub)
} by {
    if eventual_ub(a, ub) and is_subsequence_index(f) {
        eventual_ub_iff_seq_eventually_in_closed_lower_ray(a, ub)
        seq_eventually_in_real_set(closed_lower_ray(ub), a)
        seq_eventually_in_real_set_subsequence(closed_lower_ray(ub), a, f)
        seq_eventually_in_real_set(closed_lower_ray(ub), subsequence(a, f))
        eventual_ub_iff_seq_eventually_in_closed_lower_ray(subsequence(a, f), ub)
        eventual_ub(subsequence(a, f), ub)
    }
}

/// Eventual lower bounds are preserved by tail subsequences `n ↦ k + n`.
theorem eventual_lb_tail_subsequence(a: Nat -> Real, k: Nat, lb: Real) {
    eventual_lb(a, lb) implies eventual_lb(subsequence(a, k.add), lb)
} by {
    if eventual_lb(a, lb) {
        eventual_lb_iff_seq_eventually_in_closed_upper_ray(a, lb)
        seq_eventually_in_real_set(closed_upper_ray(lb), a)
        seq_eventually_in_real_set_tail_subsequence(closed_upper_ray(lb), a, k)
        seq_eventually_in_real_set(closed_upper_ray(lb), subsequence(a, k.add))
        eventual_lb_iff_seq_eventually_in_closed_upper_ray(subsequence(a, k.add), lb)
        eventual_lb(subsequence(a, k.add), lb)
    }
}

/// Eventual upper bounds are preserved by tail subsequences `n ↦ k + n`.
theorem eventual_ub_tail_subsequence(a: Nat -> Real, k: Nat, ub: Real) {
    eventual_ub(a, ub) implies eventual_ub(subsequence(a, k.add), ub)
} by {
    if eventual_ub(a, ub) {
        eventual_ub_iff_seq_eventually_in_closed_lower_ray(a, ub)
        seq_eventually_in_real_set(closed_lower_ray(ub), a)
        seq_eventually_in_real_set_tail_subsequence(closed_lower_ray(ub), a, k)
        seq_eventually_in_real_set(closed_lower_ray(ub), subsequence(a, k.add))
        eventual_ub_iff_seq_eventually_in_closed_lower_ray(subsequence(a, k.add), ub)
        eventual_ub(subsequence(a, k.add), ub)
    }
}

/// A transported eventual lower bound is below the canonical limit of an infinity-tending reindexing.
theorem compose_tends_to_infinity_eventual_lb_lte_limit(a: Nat -> Real, f: Nat -> Nat, lb: Real) {
    converges(a) and eventual_lb(a, lb) and tends_to_infinity(f)
    implies lb <= limit(compose(a, f))
} by {
    if converges(a) and eventual_lb(a, lb) and tends_to_infinity(f) {
        eventual_lb_compose_tends_to_infinity(a, f, lb)
        eventual_lb(compose(a, f), lb)
        converges_compose_tends_to_infinity(a, f)
        converges_to(compose(a, f), limit(a))
        converges_to_imp_converges(compose(a, f), limit(a))
        converges(compose(a, f))
        lb_lte_limit(compose(a, f), lb)
        lb <= limit(compose(a, f))
    }
}

/// A transported eventual upper bound is above the canonical limit of an infinity-tending reindexing.
theorem compose_tends_to_infinity_eventual_ub_limit_lte(a: Nat -> Real, f: Nat -> Nat, ub: Real) {
    converges(a) and eventual_ub(a, ub) and tends_to_infinity(f)
    implies limit(compose(a, f)) <= ub
} by {
    if converges(a) and eventual_ub(a, ub) and tends_to_infinity(f) {
        eventual_ub_compose_tends_to_infinity(a, f, ub)
        eventual_ub(compose(a, f), ub)
        converges_compose_tends_to_infinity(a, f)
        converges_to(compose(a, f), limit(a))
        converges_to_imp_converges(compose(a, f), limit(a))
        converges(compose(a, f))
        ub_imp_limit_lte(compose(a, f), ub)
        limit(compose(a, f)) <= ub
    }
}

/// A transported eventual lower bound is below the canonical limit of a finite-prefix shift.
theorem shift_add_eventual_lb_lte_limit(a: Nat -> Real, k: Nat, lb: Real) {
    converges(a) and eventual_lb(a, lb) implies lb <= limit(compose(a, k.add))
} by {
    if converges(a) and eventual_lb(a, lb) {
        eventual_lb_shift_add(a, k, lb)
        eventual_lb(compose(a, k.add), lb)
        converges_compose_add(a, k)
        converges_to(compose(a, k.add), limit(a))
        converges_to_imp_converges(compose(a, k.add), limit(a))
        converges(compose(a, k.add))
        lb_lte_limit(compose(a, k.add), lb)
        lb <= limit(compose(a, k.add))
    }
}

/// A transported eventual upper bound is above the canonical limit of a finite-prefix shift.
theorem shift_add_eventual_ub_limit_lte(a: Nat -> Real, k: Nat, ub: Real) {
    converges(a) and eventual_ub(a, ub) implies limit(compose(a, k.add)) <= ub
} by {
    if converges(a) and eventual_ub(a, ub) {
        eventual_ub_shift_add(a, k, ub)
        eventual_ub(compose(a, k.add), ub)
        converges_compose_add(a, k)
        converges_to(compose(a, k.add), limit(a))
        converges_to_imp_converges(compose(a, k.add), limit(a))
        converges(compose(a, k.add))
        ub_imp_limit_lte(compose(a, k.add), ub)
        limit(compose(a, k.add)) <= ub
    }
}

/// A transported eventual lower bound is below the canonical limit of a selected subsequence.
theorem subsequence_eventual_lb_lte_limit(a: Nat -> Real, f: Nat -> Nat, lb: Real) {
    converges(a) and eventual_lb(a, lb) and is_subsequence_index(f)
    implies lb <= limit(subsequence(a, f))
} by {
    if converges(a) and eventual_lb(a, lb) and is_subsequence_index(f) {
        eventual_lb_subsequence(a, f, lb)
        eventual_lb(subsequence(a, f), lb)
        converges_subsequence(a, f)
        converges_to(subsequence(a, f), limit(a))
        converges_to_imp_converges(subsequence(a, f), limit(a))
        converges(subsequence(a, f))
        lb_lte_limit(subsequence(a, f), lb)
        lb <= limit(subsequence(a, f))
    }
}

/// A transported eventual upper bound is above the canonical limit of a selected subsequence.
theorem subsequence_eventual_ub_limit_lte(a: Nat -> Real, f: Nat -> Nat, ub: Real) {
    converges(a) and eventual_ub(a, ub) and is_subsequence_index(f)
    implies limit(subsequence(a, f)) <= ub
} by {
    if converges(a) and eventual_ub(a, ub) and is_subsequence_index(f) {
        eventual_ub_subsequence(a, f, ub)
        eventual_ub(subsequence(a, f), ub)
        converges_subsequence(a, f)
        converges_to(subsequence(a, f), limit(a))
        converges_to_imp_converges(subsequence(a, f), limit(a))
        converges(subsequence(a, f))
        ub_imp_limit_lte(subsequence(a, f), ub)
        limit(subsequence(a, f)) <= ub
    }
}

/// A transported eventual lower bound is below the canonical limit of a tail subsequence.
theorem tail_subsequence_eventual_lb_lte_limit(a: Nat -> Real, k: Nat, lb: Real) {
    converges(a) and eventual_lb(a, lb) implies lb <= limit(subsequence(a, k.add))
} by {
    if converges(a) and eventual_lb(a, lb) {
        eventual_lb_tail_subsequence(a, k, lb)
        eventual_lb(subsequence(a, k.add), lb)
        converges_subsequence_add(a, k)
        converges_to(subsequence(a, k.add), limit(a))
        converges_to_imp_converges(subsequence(a, k.add), limit(a))
        converges(subsequence(a, k.add))
        lb_lte_limit(subsequence(a, k.add), lb)
        lb <= limit(subsequence(a, k.add))
    }
}

/// A transported eventual upper bound is above the canonical limit of a tail subsequence.
theorem tail_subsequence_eventual_ub_limit_lte(a: Nat -> Real, k: Nat, ub: Real) {
    converges(a) and eventual_ub(a, ub) implies limit(subsequence(a, k.add)) <= ub
} by {
    if converges(a) and eventual_ub(a, ub) {
        eventual_ub_tail_subsequence(a, k, ub)
        eventual_ub(subsequence(a, k.add), ub)
        converges_subsequence_add(a, k)
        converges_to(subsequence(a, k.add), limit(a))
        converges_to_imp_converges(subsequence(a, k.add), limit(a))
        converges(subsequence(a, k.add))
        ub_imp_limit_lte(subsequence(a, k.add), ub)
        limit(subsequence(a, k.add)) <= ub
    }
}
