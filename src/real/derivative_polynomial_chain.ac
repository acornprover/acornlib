from data.basic.function_algebra import pointwise_mul
from data.basic.functions import compose, identity_fn
from real.continuity_cube import cube_real, cube_real_eq_pointwise_mul_square_identity
from real.continuity_square import square_real, square_real_eq_pointwise_mul_identity
from real.derivative_basic import has_derivative_at, differentiable_at,
    identity_has_derivative_at
from real.derivative_chain import derivative_compose, differentiable_compose
from real.derivative_product import derivative_pointwise_mul,
    derivative_pointwise_square
from real.real_base import Real

/// The named square function has the product-rule derivative.
theorem square_real_has_derivative_at(x0: Real) {
    has_derivative_at(square_real, x0, x0 * Real.1 + x0 * Real.1)
} by {
    identity_has_derivative_at(x0)
    derivative_pointwise_square(identity_fn[Real], x0, Real.1)
    identity_fn[Real](x0) = x0
    has_derivative_at(pointwise_mul(identity_fn[Real], identity_fn[Real]), x0, x0 * Real.1 + x0 * Real.1)
    square_real_eq_pointwise_mul_identity
    has_derivative_at(square_real, x0, x0 * Real.1 + x0 * Real.1)
}

/// The named square function is differentiable at every point.
theorem square_real_differentiable_at(x0: Real) {
    differentiable_at(square_real, x0)
} by {
    square_real_has_derivative_at(x0)
    exists(d: Real) {
        has_derivative_at(square_real, x0, d)
    }
}

/// The named cube function has the iterated product-rule derivative.
theorem cube_real_has_derivative_at(x0: Real) {
    has_derivative_at(
        cube_real,
        x0,
        square_real(x0) * Real.1 + x0 * (x0 * Real.1 + x0 * Real.1)
    )
} by {
    square_real_has_derivative_at(x0)
    identity_has_derivative_at(x0)
    derivative_pointwise_mul(square_real, identity_fn[Real], x0, x0 * Real.1 + x0 * Real.1, Real.1)
    identity_fn[Real](x0) = x0
    has_derivative_at(
        pointwise_mul(square_real, identity_fn[Real]),
        x0,
        square_real(x0) * Real.1 + x0 * (x0 * Real.1 + x0 * Real.1)
    )
    cube_real_eq_pointwise_mul_square_identity
    has_derivative_at(
        cube_real,
        x0,
        square_real(x0) * Real.1 + x0 * (x0 * Real.1 + x0 * Real.1)
    )
}

/// The named cube function is differentiable at every point.
theorem cube_real_differentiable_at(x0: Real) {
    differentiable_at(cube_real, x0)
} by {
    cube_real_has_derivative_at(x0)
    exists(d: Real) {
        has_derivative_at(cube_real, x0, d)
    }
}

/// Composing the square function after a differentiable function follows the chain rule.
theorem derivative_square_real_compose(f: Real -> Real, x0: Real, d: Real) {
    has_derivative_at(f, x0, d)
    implies has_derivative_at(compose(square_real, f), x0, (f(x0) * Real.1 + f(x0) * Real.1) * d)
} by {
    square_real_has_derivative_at(f(x0))
    derivative_compose(f, square_real, x0, d, f(x0) * Real.1 + f(x0) * Real.1)
    has_derivative_at(compose(square_real, f), x0, (f(x0) * Real.1 + f(x0) * Real.1) * d)
}

/// Composing the cube function after a differentiable function follows the chain rule.
theorem derivative_cube_real_compose(f: Real -> Real, x0: Real, d: Real) {
    has_derivative_at(f, x0, d)
    implies has_derivative_at(
        compose(cube_real, f),
        x0,
        (square_real(f(x0)) * Real.1 + f(x0) * (f(x0) * Real.1 + f(x0) * Real.1)) * d
    )
} by {
    cube_real_has_derivative_at(f(x0))
    derivative_compose(
        f,
        cube_real,
        x0,
        d,
        square_real(f(x0)) * Real.1 + f(x0) * (f(x0) * Real.1 + f(x0) * Real.1)
    )
    has_derivative_at(
        compose(cube_real, f),
        x0,
        (square_real(f(x0)) * Real.1 + f(x0) * (f(x0) * Real.1 + f(x0) * Real.1)) * d
    )
}

/// Differentiability is preserved by composing the square function after a differentiable function.
theorem differentiable_square_real_compose(f: Real -> Real, x0: Real) {
    differentiable_at(f, x0) implies differentiable_at(compose(square_real, f), x0)
} by {
    square_real_differentiable_at(f(x0))
    differentiable_compose(f, square_real, x0)
    differentiable_at(compose(square_real, f), x0)
}

/// Differentiability is preserved by composing the cube function after a differentiable function.
theorem differentiable_cube_real_compose(f: Real -> Real, x0: Real) {
    differentiable_at(f, x0) implies differentiable_at(compose(cube_real, f), x0)
} by {
    cube_real_differentiable_at(f(x0))
    differentiable_compose(f, cube_real, x0)
    differentiable_at(compose(cube_real, f), x0)
}
