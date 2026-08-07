from data.basic.function_algebra import pointwise_add, pointwise_neg
from real.continuity_base import Real, continuous, continuous_at, continuous_condition
from real.continuity_composition import continuous_at_delta, continuous_at_intro,
    continuous_intro
from real.real_seq import add_close, close_and_lt_imp_close, eps_lt_half,
    eps_smaller_than_both, neg_is_close

/// Pointwise negation preserves continuity at a point.
theorem continuous_at_pointwise_neg(f: Real -> Real, x: Real) {
    continuous_at(f, x) implies continuous_at(pointwise_neg(f), x)
} by {
    forall(eps: Real) {
        if eps.is_positive {
            continuous_at_delta(f, x, eps)
            let delta: Real satisfy {
                delta.is_positive and continuous_condition(f, x, delta, eps)
            }
            continuous_condition(f, x, delta, eps) = forall(y: Real) {
                y.is_close(x, delta) implies f(y).is_close(f(x), eps)
            }
            forall(y: Real) {
                if y.is_close(x, delta) {
                    f(y).is_close(f(x), eps)
                    neg_is_close(f(y), f(x), eps)
                    (-f(y)).is_close(-f(x), eps)
                    pointwise_neg(f, y) = -f(y)
                    pointwise_neg(f, x) = -f(x)
                    pointwise_neg(f, y).is_close(pointwise_neg(f, x), eps)
                }
            }
            continuous_condition(pointwise_neg(f), x, delta, eps)
            delta.is_positive and continuous_condition(pointwise_neg(f), x, delta, eps)
            exists(delta2: Real) {
                delta2.is_positive and continuous_condition(pointwise_neg(f), x, delta2, eps)
            }
        }
    }
    continuous_at_intro(pointwise_neg(f), x)
    continuous_at(pointwise_neg(f), x)
}

/// Pointwise negation preserves continuous real functions.
theorem continuous_pointwise_neg(f: Real -> Real) {
    continuous(f) implies continuous(pointwise_neg(f))
} by {
    continuous(f) = forall(x: Real) {
        continuous_at(f, x)
    }
    forall(x: Real) {
        continuous_at(f, x)
        continuous_at_pointwise_neg(f, x)
        continuous_at(pointwise_neg(f), x)
    }
    continuous_intro(pointwise_neg(f))
    continuous(pointwise_neg(f))
}

/// Local continuity conditions are preserved by pointwise addition.
theorem continuous_condition_pointwise_add(
    f: Real -> Real, g: Real -> Real, x: Real,
    delta: Real, delta_f: Real, delta_g: Real, eps2: Real, eps: Real
) {
    delta < delta_f and delta < delta_g and eps2 + eps2 < eps
    and continuous_condition(f, x, delta_f, eps2)
    and continuous_condition(g, x, delta_g, eps2)
    implies continuous_condition(pointwise_add(f, g), x, delta, eps)
} by {
    continuous_condition(f, x, delta_f, eps2) = forall(y: Real) {
        y.is_close(x, delta_f) implies f(y).is_close(f(x), eps2)
    }
    continuous_condition(g, x, delta_g, eps2) = forall(y: Real) {
        y.is_close(x, delta_g) implies g(y).is_close(g(x), eps2)
    }
    forall(y: Real) {
        if y.is_close(x, delta) {
            close_and_lt_imp_close(y, x, delta, delta_f)
            close_and_lt_imp_close(y, x, delta, delta_g)
            y.is_close(x, delta_f)
            y.is_close(x, delta_g)
            f(y).is_close(f(x), eps2)
            g(y).is_close(g(x), eps2)
            add_close(f(y), g(y), f(x), g(x), eps2, eps2)
            (f(y) + g(y)).is_close(f(x) + g(x), eps2 + eps2)
            close_and_lt_imp_close(f(y) + g(y), f(x) + g(x), eps2 + eps2, eps)
            pointwise_add(f, g, y) = f(y) + g(y)
            pointwise_add(f, g, x) = f(x) + g(x)
            pointwise_add(f, g, y).is_close(pointwise_add(f, g, x), eps)
        }
    }
}

/// Pointwise addition preserves continuity at a point.
theorem continuous_at_pointwise_add(f: Real -> Real, g: Real -> Real, x: Real) {
    continuous_at(f, x) and continuous_at(g, x) implies continuous_at(pointwise_add(f, g), x)
} by {
    forall(eps: Real) {
        if eps.is_positive {
            eps_lt_half(eps)
            let eps2: Real satisfy {
                eps2.is_positive and eps2 + eps2 < eps
            }
            continuous_at_delta(f, x, eps2)
            let delta_f: Real satisfy {
                delta_f.is_positive and continuous_condition(f, x, delta_f, eps2)
            }
            continuous_at_delta(g, x, eps2)
            let delta_g: Real satisfy {
                delta_g.is_positive and continuous_condition(g, x, delta_g, eps2)
            }
            eps_smaller_than_both(delta_f, delta_g)
            let delta: Real satisfy {
                delta.is_positive and delta < delta_f and delta < delta_g
            }
            continuous_condition_pointwise_add(f, g, x, delta, delta_f, delta_g, eps2, eps)
            continuous_condition(pointwise_add(f, g), x, delta, eps)
            delta.is_positive and continuous_condition(pointwise_add(f, g), x, delta, eps)
            exists(delta2: Real) {
                delta2.is_positive and continuous_condition(pointwise_add(f, g), x, delta2, eps)
            }
        }
    }
    continuous_at(pointwise_add(f, g), x) = forall(eps2: Real) {
        eps2.is_positive implies exists(delta2: Real) {
            delta2.is_positive and continuous_condition(pointwise_add(f, g), x, delta2, eps2)
        }
    }
    if not continuous_at(pointwise_add(f, g), x) {
        not forall(eps2: Real) {
            eps2.is_positive implies exists(delta2: Real) {
                delta2.is_positive and continuous_condition(pointwise_add(f, g), x, delta2, eps2)
            }
        }
        let bad_eps: Real satisfy {
            bad_eps.is_positive and forall(delta2: Real) {
                not (delta2.is_positive and continuous_condition(pointwise_add(f, g), x, delta2, bad_eps))
            }
        }
        exists(delta2: Real) {
            delta2.is_positive and continuous_condition(pointwise_add(f, g), x, delta2, bad_eps)
        }
        false
    }
    continuous_at(pointwise_add(f, g), x)
}

/// Pointwise addition preserves continuous real functions.
theorem continuous_pointwise_add(f: Real -> Real, g: Real -> Real) {
    continuous(f) and continuous(g) implies continuous(pointwise_add(f, g))
} by {
    continuous(f) = forall(x: Real) {
        continuous_at(f, x)
    }
    continuous(g) = forall(x: Real) {
        continuous_at(g, x)
    }
    forall(x: Real) {
        continuous_at(f, x)
        continuous_at(g, x)
        continuous_at_pointwise_add(f, g, x)
        continuous_at(pointwise_add(f, g), x)
    }
    continuous_intro(pointwise_add(f, g))
    continuous(pointwise_add(f, g))
}

/// Pointwise subtraction preserves continuity at a point.
theorem continuous_at_pointwise_sub(f: Real -> Real, g: Real -> Real, x: Real) {
    continuous_at(f, x) and continuous_at(g, x)
    implies continuous_at(pointwise_add(f, pointwise_neg(g)), x)
} by {
    continuous_at_pointwise_neg(g, x)
    continuous_at(pointwise_neg(g), x)
    continuous_at_pointwise_add(f, pointwise_neg(g), x)
    continuous_at(pointwise_add(f, pointwise_neg(g)), x)
}

/// Pointwise subtraction preserves continuous real functions.
theorem continuous_pointwise_sub(f: Real -> Real, g: Real -> Real) {
    continuous(f) and continuous(g) implies continuous(pointwise_add(f, pointwise_neg(g)))
} by {
    continuous_pointwise_neg(g)
    continuous(pointwise_neg(g))
    continuous_pointwise_add(f, pointwise_neg(g))
    continuous(pointwise_add(f, pointwise_neg(g)))
}
