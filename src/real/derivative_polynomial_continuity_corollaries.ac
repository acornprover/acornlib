/// Continuity corollaries for named polynomial derivative rules.

from data.basic.function_algebra import pointwise_mul
from data.basic.functions import compose
from real.continuity_base import continuous_at
from real.continuity_cube import cube_real
from real.continuity_square import square_real
from real.derivative_basic import differentiable_at, has_derivative_at
from real.derivative_continuity import derivative_continuous_at,
    differentiable_continuous_at
from real.derivative_polynomial_chain import derivative_square_real_compose,
    differentiable_square_real_compose, derivative_cube_real_compose,
    differentiable_cube_real_compose
from real.derivative_polynomial_product import derivative_square_real_mul,
    derivative_mul_square_real, derivative_cube_real_mul, derivative_mul_cube_real,
    differentiable_square_real_mul, differentiable_mul_square_real,
    differentiable_cube_real_mul, differentiable_mul_cube_real
from real.real_base import Real

/// A square composition with a derivative at the point is continuous there.
theorem derivative_square_real_compose_continuous_at(f: Real -> Real, x0: Real, d: Real) {
    has_derivative_at(f, x0, d) implies continuous_at(compose(square_real, f), x0)
} by {
    if has_derivative_at(f, x0, d) {
        derivative_square_real_compose(f, x0, d)
        has_derivative_at(
            compose(square_real, f),
            x0,
            (f(x0) * Real.1 + f(x0) * Real.1) * d
        )
        derivative_continuous_at(
            compose(square_real, f),
            x0,
            (f(x0) * Real.1 + f(x0) * Real.1) * d
        )
        continuous_at(compose(square_real, f), x0)
    }
}

/// A square composition of a differentiable function is continuous at the point.
theorem differentiable_square_real_compose_continuous_at(f: Real -> Real, x0: Real) {
    differentiable_at(f, x0) implies continuous_at(compose(square_real, f), x0)
} by {
    if differentiable_at(f, x0) {
        differentiable_square_real_compose(f, x0)
        differentiable_at(compose(square_real, f), x0)
        differentiable_continuous_at(compose(square_real, f), x0)
        continuous_at(compose(square_real, f), x0)
    }
}

/// A cube composition with a derivative at the point is continuous there.
theorem derivative_cube_real_compose_continuous_at(f: Real -> Real, x0: Real, d: Real) {
    has_derivative_at(f, x0, d) implies continuous_at(compose(cube_real, f), x0)
} by {
    if has_derivative_at(f, x0, d) {
        derivative_cube_real_compose(f, x0, d)
        has_derivative_at(
            compose(cube_real, f),
            x0,
            (square_real(f(x0)) * Real.1 + f(x0) * (f(x0) * Real.1 + f(x0) * Real.1)) * d
        )
        derivative_continuous_at(
            compose(cube_real, f),
            x0,
            (square_real(f(x0)) * Real.1 + f(x0) * (f(x0) * Real.1 + f(x0) * Real.1)) * d
        )
        continuous_at(compose(cube_real, f), x0)
    }
}

/// A cube composition of a differentiable function is continuous at the point.
theorem differentiable_cube_real_compose_continuous_at(f: Real -> Real, x0: Real) {
    differentiable_at(f, x0) implies continuous_at(compose(cube_real, f), x0)
} by {
    if differentiable_at(f, x0) {
        differentiable_cube_real_compose(f, x0)
        differentiable_at(compose(cube_real, f), x0)
        differentiable_continuous_at(compose(cube_real, f), x0)
        continuous_at(compose(cube_real, f), x0)
    }
}

/// Multiplying a differentiable function by the square function on the left is continuous at the point.
theorem derivative_square_real_mul_continuous_at(f: Real -> Real, x0: Real, d: Real) {
    has_derivative_at(f, x0, d) implies continuous_at(pointwise_mul(square_real, f), x0)
} by {
    if has_derivative_at(f, x0, d) {
        derivative_square_real_mul(f, x0, d)
        has_derivative_at(
            pointwise_mul(square_real, f),
            x0,
            square_real(x0) * d + f(x0) * (x0 * Real.1 + x0 * Real.1)
        )
        derivative_continuous_at(
            pointwise_mul(square_real, f),
            x0,
            square_real(x0) * d + f(x0) * (x0 * Real.1 + x0 * Real.1)
        )
        continuous_at(pointwise_mul(square_real, f), x0)
    }
}

/// Multiplying a differentiable function by the square function on the right is continuous at the point.
theorem derivative_mul_square_real_continuous_at(f: Real -> Real, x0: Real, d: Real) {
    has_derivative_at(f, x0, d) implies continuous_at(pointwise_mul(f, square_real), x0)
} by {
    if has_derivative_at(f, x0, d) {
        derivative_mul_square_real(f, x0, d)
        has_derivative_at(
            pointwise_mul(f, square_real),
            x0,
            f(x0) * (x0 * Real.1 + x0 * Real.1) + square_real(x0) * d
        )
        derivative_continuous_at(
            pointwise_mul(f, square_real),
            x0,
            f(x0) * (x0 * Real.1 + x0 * Real.1) + square_real(x0) * d
        )
        continuous_at(pointwise_mul(f, square_real), x0)
    }
}

/// Multiplying a differentiable function by the cube function on the left is continuous at the point.
theorem derivative_cube_real_mul_continuous_at(f: Real -> Real, x0: Real, d: Real) {
    has_derivative_at(f, x0, d) implies continuous_at(pointwise_mul(cube_real, f), x0)
} by {
    if has_derivative_at(f, x0, d) {
        derivative_cube_real_mul(f, x0, d)
        has_derivative_at(
            pointwise_mul(cube_real, f),
            x0,
            cube_real(x0) * d + f(x0) * (square_real(x0) * Real.1 + x0 * (x0 * Real.1 + x0 * Real.1))
        )
        derivative_continuous_at(
            pointwise_mul(cube_real, f),
            x0,
            cube_real(x0) * d + f(x0) * (square_real(x0) * Real.1 + x0 * (x0 * Real.1 + x0 * Real.1))
        )
        continuous_at(pointwise_mul(cube_real, f), x0)
    }
}

/// Multiplying a differentiable function by the cube function on the right is continuous at the point.
theorem derivative_mul_cube_real_continuous_at(f: Real -> Real, x0: Real, d: Real) {
    has_derivative_at(f, x0, d) implies continuous_at(pointwise_mul(f, cube_real), x0)
} by {
    if has_derivative_at(f, x0, d) {
        derivative_mul_cube_real(f, x0, d)
        has_derivative_at(
            pointwise_mul(f, cube_real),
            x0,
            f(x0) * (square_real(x0) * Real.1 + x0 * (x0 * Real.1 + x0 * Real.1)) + cube_real(x0) * d
        )
        derivative_continuous_at(
            pointwise_mul(f, cube_real),
            x0,
            f(x0) * (square_real(x0) * Real.1 + x0 * (x0 * Real.1 + x0 * Real.1)) + cube_real(x0) * d
        )
        continuous_at(pointwise_mul(f, cube_real), x0)
    }
}

/// Multiplying a differentiable function by the square function on the left is continuous at the point.
theorem differentiable_square_real_mul_continuous_at(f: Real -> Real, x0: Real) {
    differentiable_at(f, x0) implies continuous_at(pointwise_mul(square_real, f), x0)
} by {
    if differentiable_at(f, x0) {
        differentiable_square_real_mul(f, x0)
        differentiable_at(pointwise_mul(square_real, f), x0)
        differentiable_continuous_at(pointwise_mul(square_real, f), x0)
        continuous_at(pointwise_mul(square_real, f), x0)
    }
}

/// Multiplying a differentiable function by the square function on the right is continuous at the point.
theorem differentiable_mul_square_real_continuous_at(f: Real -> Real, x0: Real) {
    differentiable_at(f, x0) implies continuous_at(pointwise_mul(f, square_real), x0)
} by {
    if differentiable_at(f, x0) {
        differentiable_mul_square_real(f, x0)
        differentiable_at(pointwise_mul(f, square_real), x0)
        differentiable_continuous_at(pointwise_mul(f, square_real), x0)
        continuous_at(pointwise_mul(f, square_real), x0)
    }
}

/// Multiplying a differentiable function by the cube function on the left is continuous at the point.
theorem differentiable_cube_real_mul_continuous_at(f: Real -> Real, x0: Real) {
    differentiable_at(f, x0) implies continuous_at(pointwise_mul(cube_real, f), x0)
} by {
    if differentiable_at(f, x0) {
        differentiable_cube_real_mul(f, x0)
        differentiable_at(pointwise_mul(cube_real, f), x0)
        differentiable_continuous_at(pointwise_mul(cube_real, f), x0)
        continuous_at(pointwise_mul(cube_real, f), x0)
    }
}

/// Multiplying a differentiable function by the cube function on the right is continuous at the point.
theorem differentiable_mul_cube_real_continuous_at(f: Real -> Real, x0: Real) {
    differentiable_at(f, x0) implies continuous_at(pointwise_mul(f, cube_real), x0)
} by {
    if differentiable_at(f, x0) {
        differentiable_mul_cube_real(f, x0)
        differentiable_at(pointwise_mul(f, cube_real), x0)
        differentiable_continuous_at(pointwise_mul(f, cube_real), x0)
        continuous_at(pointwise_mul(f, cube_real), x0)
    }
}
