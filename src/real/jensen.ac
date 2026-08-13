/// Jensen's inequality for convex functions.
///
/// For a convex function f, nonnegative weights w(0), ..., w(n-1) summing to
/// one, and points x(0), ..., x(n-1) in the interval of convexity, the value
/// of f at the weighted average of the points is at most the weighted average
/// of the values: f(sum_i w(i) x(i)) <= sum_i w(i) f(x(i)).  The proof
/// inducts on the number of terms: the induction step splits the final weight
/// off, applies the two-point weighted inequality to the rescaled average of
/// the earlier points and the final point, and applies the induction
/// hypothesis to the rescaled weights.

from nat import Nat, alt_induction, lt_suc, lt_imp_lt_suc, not_lt_zero, lt_or_lte,
    lt_imp_lte_suc
from list import partial, partial_split_last, partial_zero, partial_scalar_mul,
    partial_pointwise_eq
from algebra.semigroup import mul_fn
from order import lte_refl, lte_trans, lte_antisymm, lt_imp_lte, lt_imp_ne,
    lt_of_lte_of_ne
from real.real_base import add_comm, add_assoc, add_zero_left, add_zero_right,
    add_neg_eq_zero, lte_add_right, lte_add_left, add_lte_add, lt_add_right,
    gt_zero_imp_pos
from real.real_ring import mul_zero_left, mul_one_left, mul_one_right, mul_nonneg,
    real_mul_comm, mul_assoc, mul_distrib_left
from real.real_field import Real, mul_inverse, zero_is_different_than_one
from ordered_field import inverse_of_positive_is_positive, mul_le_mul_of_nonneg_right
from real.real_series import is_lower_bound, partial_nonneg, partial_all_zeros
from real.convex import convex_on, convex_weighted_inequality
from real.cauchy_schwarz import pointwise_product
from real.am_gm import add_nonneg_eq_zero_left, add_nonneg_eq_zero_right,
    partial_nonneg_eq_zero_imp_each

numerals Real

/// The pointwise image of a point sequence under a function.
define pointwise_apply(f: Real -> Real, x: Nat -> Real, i: Nat) -> Real {
    f(x(i))
}

/// The weighted point sum sum_{i<n} w(i) * x(i).
define weighted_point_sum(w: Nat -> Real, x: Nat -> Real, n: Nat) -> Real {
    partial(pointwise_product(w, x), n)
}

/// The weighted sum of values sum_{i<n} w(i) * f(x(i)).
define weighted_value_sum(f: Real -> Real, w: Nat -> Real, x: Nat -> Real, n: Nat) -> Real {
    partial(pointwise_product(w, pointwise_apply(f, x)), n)
}

/// The weight at index i scaled by the inverse of c: c.inverse * w(i).
define scaled_weight(c: Real, w: Nat -> Real, i: Nat) -> Real {
    c.inverse * w(i)
}

/// True if the point x(i) lies in the interval [a, b].
define jensen_point_in_interval_at(a: Real, b: Real, x: Nat -> Real, i: Nat) -> Bool {
    a <= x(i) and x(i) <= b
}

/// True if every point below the bound lies in the interval [a, b].
define jensen_points_in_interval(a: Real, b: Real, x: Nat -> Real, n: Nat) -> Bool {
    forall(i: Nat) {
        i < n implies jensen_point_in_interval_at(a, b, x, i)
    }
}

/// A point below the bound lies in the interval.
theorem jensen_points_in_interval_apply(a: Real, b: Real, x: Nat -> Real, n: Nat, i: Nat) {
    jensen_points_in_interval(a, b, x, n) and i < n implies a <= x(i) and x(i) <= b
} by {
    if jensen_points_in_interval(a, b, x, n) and i < n {
        jensen_points_in_interval(a, b, x, n) =
            forall(j: Nat) { j < n implies jensen_point_in_interval_at(a, b, x, j) }
        i < n implies jensen_point_in_interval_at(a, b, x, i)
        jensen_point_in_interval_at(a, b, x, i)
        jensen_point_in_interval_at(a, b, x, i) = (a <= x(i) and x(i) <= b)
        a <= x(i) and x(i) <= b
    }
}

/// Points in the interval below a larger bound stay in the interval below the
/// smaller bound.
theorem jensen_points_in_interval_restrict(a: Real, b: Real, x: Nat -> Real, n: Nat) {
    jensen_points_in_interval(a, b, x, n.suc) implies jensen_points_in_interval(a, b, x, n)
} by {
    if jensen_points_in_interval(a, b, x, n.suc) {
        forall(i: Nat) {
            if i < n {
                jensen_points_in_interval(a, b, x, n.suc) =
                    forall(j: Nat) { j < n.suc implies jensen_point_in_interval_at(a, b, x, j) }
                lt_imp_lt_suc(i, n)
                i < n.suc
                i < n.suc implies jensen_point_in_interval_at(a, b, x, i)
                jensen_point_in_interval_at(a, b, x, i)
                jensen_point_in_interval_at(a, b, x, i) = (a <= x(i) and x(i) <= b)
                a <= x(i) and x(i) <= b
            }
        }
        jensen_points_in_interval(a, b, x, n) =
            forall(j: Nat) { j < n implies jensen_point_in_interval_at(a, b, x, j) }
        jensen_points_in_interval(a, b, x, n)
    }
}

/// A zero partial sum of nonnegative weights forces each weight in the range
/// to be zero.
theorem jensen_weight_zero_of_partial_zero(w: Nat -> Real, n: Nat, i: Nat) {
    (forall(j: Nat) { Real.0 <= w(j) }) and partial(w, n) = Real.0 and i < n implies w(i) = Real.0
} by {
    define statement(k: Nat) -> Bool {
        forall(w0: Nat -> Real, j: Nat) {
            (forall(h: Nat) { Real.0 <= w0(h) }) and partial(w0, k) = Real.0 and j < k implies w0(j) = Real.0
        }
    }

    forall(w0: Nat -> Real, j: Nat) {
        if (forall(h: Nat) { Real.0 <= w0(h) }) and partial(w0, Nat.0) = Real.0 and j < Nat.0 {
            not_lt_zero(j)
            false
        }
    }
    statement(Nat.0)

    forall(k: Nat) {
        if statement(k) {
            forall(w0: Nat -> Real, j: Nat) {
                if (forall(h: Nat) { Real.0 <= w0(h) }) and partial(w0, k.suc) = Real.0 and j < k.suc {
                    partial_split_last(w0, k)
                    partial(w0, k.suc) = partial(w0, k) + w0(k)
                    partial(w0, k) + w0(k) = Real.0
                    is_lower_bound(w0, Real.0) = forall(h: Nat) { Real.0 <= w0(h) }
                    is_lower_bound(w0, Real.0)
                    partial_nonneg(w0, k)
                    partial(w0, k) >= Real.0
                    Real.0 <= partial(w0, k)
                    Real.0 <= w0(k)
                    w0(k) >= Real.0
                    add_nonneg_eq_zero_left(partial(w0, k), w0(k))
                    partial(w0, k) = Real.0
                    add_nonneg_eq_zero_right(partial(w0, k), w0(k))
                    w0(k) = Real.0
                    if j < k {
                        statement(k) = forall(w1: Nat -> Real, j1: Nat) {
                            (forall(h: Nat) { Real.0 <= w1(h) }) and partial(w1, k) = Real.0 and j1 < k implies w1(j1) = Real.0
                        }
                        w0(j) = Real.0
                    } else {
                        not j < k
                        lt_or_lte(j, k)
                        if j < k {
                            false
                        }
                        k <= j
                        lt_imp_lte_suc(j, k)
                        j <= k
                        lte_antisymm[Nat](j, k)
                        j = k
                        w0(j) = w0(k)
                        w0(j) = Real.0
                    }
                }
            }
            statement(k.suc)
        }
    }

    statement(Nat.0) and forall(k: Nat) { statement(k) implies statement(k.suc) }
    alt_induction(statement)
    forall(k: Nat) { statement(k) }
    statement(n)
    statement(n) = forall(w0: Nat -> Real, j: Nat) {
        (forall(h: Nat) { Real.0 <= w0(h) }) and partial(w0, n) = Real.0 and j < n implies w0(j) = Real.0
    }
    (forall(j: Nat) { Real.0 <= w(j) }) and partial(w, n) = Real.0 and i < n
    w(i) = Real.0
}

/// A convex combination of two points of a closed interval lies in the
/// interval.
theorem convex_combination_in_closed_interval(a: Real, b: Real, u: Real, v: Real, t: Real) {
    a <= u and u <= b and a <= v and v <= b and Real.0 <= t and t <= Real.1
    implies a <= (Real.1 - t) * u + t * v and (Real.1 - t) * u + t * v <= b
} by {
    if a <= u and u <= b and a <= v and v <= b and Real.0 <= t and t <= Real.1 {
        lte_add_right(t, Real.1, -t)
        t + -t <= Real.1 + -t
        add_neg_eq_zero(t)
        t + -t = Real.0
        Real.0 <= Real.1 + -t
        Real.1 - t = Real.1 + -t
        Real.0 <= Real.1 - t
        mul_le_mul_of_nonneg_right[Real](a, u, Real.1 - t)
        a * (Real.1 - t) <= u * (Real.1 - t)
        mul_le_mul_of_nonneg_right[Real](a, v, t)
        a * t <= v * t
        add_lte_add(a * (Real.1 - t), u * (Real.1 - t), a * t, v * t)
        a * (Real.1 - t) + a * t <= u * (Real.1 - t) + v * t
        mul_distrib_left(a, Real.1 - t, t)
        a * ((Real.1 - t) + t) = a * (Real.1 - t) + a * t
        add_assoc(Real.1, -t, t)
        (Real.1 + -t) + t = Real.1 + (-t + t)
        add_neg_eq_zero(t)
        -t + t = Real.0
        Real.1 + (-t + t) = Real.1 + Real.0
        add_zero_right(Real.1)
        Real.1 + Real.0 = Real.1
        (Real.1 + -t) + t = Real.1
        Real.1 - t = Real.1 + -t
        (Real.1 - t) + t = Real.1
        a * ((Real.1 - t) + t) = a * Real.1
        mul_one_right(a)
        a * Real.1 = a
        a * (Real.1 - t) + a * t = a
        real_mul_comm(u, Real.1 - t)
        u * (Real.1 - t) = (Real.1 - t) * u
        a <= (Real.1 - t) * u + t * v
        mul_le_mul_of_nonneg_right[Real](u, b, Real.1 - t)
        u * (Real.1 - t) <= b * (Real.1 - t)
        mul_le_mul_of_nonneg_right[Real](v, b, t)
        v * t <= b * t
        add_lte_add(u * (Real.1 - t), b * (Real.1 - t), v * t, b * t)
        u * (Real.1 - t) + v * t <= b * (Real.1 - t) + b * t
        mul_distrib_left(b, Real.1 - t, t)
        b * ((Real.1 - t) + t) = b * (Real.1 - t) + b * t
        b * ((Real.1 - t) + t) = b * Real.1
        mul_one_right(b)
        b * Real.1 = b
        b * (Real.1 - t) + b * t = b
        (Real.1 - t) * u + t * v <= b
        a <= (Real.1 - t) * u + t * v and (Real.1 - t) * u + t * v <= b
    }
}

/// The partial sum of the weights splits off the last weight:
/// partial(w, n) = 1 - w(n).
theorem jensen_weights_split(w: Nat -> Real, n: Nat) {
    partial(w, n.suc) = Real.1 implies partial(w, n) = Real.1 - w(n)
} by {
    if partial(w, n.suc) = Real.1 {
        partial_split_last(w, n)
        partial(w, n.suc) = partial(w, n) + w(n)
        partial(w, n) + w(n) = Real.1
        add_assoc(partial(w, n), w(n), -w(n))
        (partial(w, n) + w(n)) + -w(n) = partial(w, n) + (w(n) + -w(n))
        add_neg_eq_zero(w(n))
        w(n) + -w(n) = Real.0
        partial(w, n) + (w(n) + -w(n)) = partial(w, n) + Real.0
        add_zero_right(partial(w, n))
        partial(w, n) + Real.0 = partial(w, n)
        (partial(w, n) + w(n)) + -w(n) = partial(w, n)
        (partial(w, n) + w(n)) + -w(n) = Real.1 + -w(n)
        Real.1 - w(n) = Real.1 + -w(n)
        partial(w, n) = Real.1 - w(n)
    }
}

/// A scaled weight with a positive scale is nonnegative when the weight is.
theorem scaled_weight_nonneg(c: Real, w: Nat -> Real, i: Nat) {
    Real.0 < c and Real.0 <= w(i) implies Real.0 <= scaled_weight(c, w, i)
} by {
    if Real.0 < c and Real.0 <= w(i) {
        gt_zero_imp_pos(c)
        c.is_positive
        inverse_of_positive_is_positive[Real](c)
        Real.0 < c.inverse
        lt_imp_lte(Real.0, c.inverse)
        Real.0 <= c.inverse
        c.inverse >= Real.0
        Real.0 <= w(i)
        w(i) >= Real.0
        mul_nonneg(c.inverse, w(i))
        c.inverse * w(i) >= Real.0
        Real.0 <= c.inverse * w(i)
        scaled_weight(c, w, i) = c.inverse * w(i)
        Real.0 <= scaled_weight(c, w, i)
    }
}

/// The partial sum of scaled weights is the scaled partial sum.
theorem partial_scaled_weight(c: Real, w: Nat -> Real, n: Nat) {
    partial(scaled_weight(c, w), n) = c.inverse * partial(w, n)
} by {
    forall(i: Nat) {
        scaled_weight(c, w, i) = c.inverse * w(i)
        mul_fn(c.inverse, w, i) = c.inverse * w(i)
        scaled_weight(c, w, i) = mul_fn(c.inverse, w, i)
    }
    partial_pointwise_eq(scaled_weight(c, w), mul_fn(c.inverse, w), n)
    partial(scaled_weight(c, w), n) = partial(mul_fn(c.inverse, w), n)
    partial_scalar_mul[Real](c.inverse, w, n)
    c.inverse * partial(w, n) = partial(mul_fn(c.inverse, w), n)
    partial(scaled_weight(c, w), n) = c.inverse * partial(w, n)
}

/// Scaling the weights by the inverse of a nonzero c scales the weighted sum:
/// sum_i w(i) * y(i) = c * sum_i (c.inverse * w(i)) * y(i).
theorem partial_weighted_scaled(c: Real, w: Nat -> Real, y: Nat -> Real, n: Nat) {
    c != Real.0 implies
    partial(pointwise_product(w, y), n) = c * partial(pointwise_product(scaled_weight(c, w), y), n)
} by {
    if c != Real.0 {
        forall(i: Nat) {
            pointwise_product(w, y, i) = w(i) * y(i)
            pointwise_product(scaled_weight(c, w), y, i) = scaled_weight(c, w, i) * y(i)
            scaled_weight(c, w, i) = c.inverse * w(i)
            mul_assoc(c, c.inverse, w(i))
            c * (c.inverse * w(i)) = (c * c.inverse) * w(i)
            mul_inverse(c)
            c * c.inverse = Real.1
            (c * c.inverse) * w(i) = Real.1 * w(i)
            mul_one_left(w(i))
            Real.1 * w(i) = w(i)
            c * (c.inverse * w(i)) = w(i)
            pointwise_product(scaled_weight(c, w), y, i) = c.inverse * (w(i) * y(i))
            mul_assoc(c.inverse, w(i), y(i))
            (c.inverse * w(i)) * y(i) = c.inverse * (w(i) * y(i))
            pointwise_product(scaled_weight(c, w), y, i) = c.inverse * pointwise_product(w, y, i)
            mul_assoc(c, c.inverse, pointwise_product(w, y, i))
            c * (c.inverse * pointwise_product(w, y, i)) = (c * c.inverse) * pointwise_product(w, y, i)
            (c * c.inverse) * pointwise_product(w, y, i) = Real.1 * pointwise_product(w, y, i)
            mul_one_left(pointwise_product(w, y, i))
            Real.1 * pointwise_product(w, y, i) = pointwise_product(w, y, i)
            c * (c.inverse * pointwise_product(w, y, i)) = pointwise_product(w, y, i)
            c * pointwise_product(scaled_weight(c, w), y, i) = pointwise_product(w, y, i)
            pointwise_product(w, y, i) = c * pointwise_product(scaled_weight(c, w), y, i)
            mul_fn(c, pointwise_product(scaled_weight(c, w), y), i) = c * pointwise_product(scaled_weight(c, w), y, i)
            pointwise_product(w, y, i) = mul_fn(c, pointwise_product(scaled_weight(c, w), y), i)
        }
        partial_pointwise_eq(pointwise_product(w, y), mul_fn(c, pointwise_product(scaled_weight(c, w), y)), n)
        partial(pointwise_product(w, y), n) = partial(mul_fn(c, pointwise_product(scaled_weight(c, w), y)), n)
        partial_scalar_mul[Real](c, pointwise_product(scaled_weight(c, w), y), n)
        c * partial(pointwise_product(scaled_weight(c, w), y), n) = partial(mul_fn(c, pointwise_product(scaled_weight(c, w), y)), n)
        partial(pointwise_product(w, y), n) = c * partial(pointwise_product(scaled_weight(c, w), y), n)
    }
}

/// The induction claim for Jensen's inequality at bound k and one pair of
/// sequences: nonnegative weights summing to one and points in the interval
/// give the interval bounds and the Jensen inequality.
define jensen_claim_at(f: Real -> Real, a: Real, b: Real, k: Nat, w0: Nat -> Real, x0: Nat -> Real) -> Bool {
    (forall(i: Nat) { Real.0 <= w0(i) }) and partial(w0, k) = Real.1 and
    jensen_points_in_interval(a, b, x0, k)
    implies (a <= weighted_point_sum(w0, x0, k) and weighted_point_sum(w0, x0, k) <= b) and
        f(weighted_point_sum(w0, x0, k)) <= weighted_value_sum(f, w0, x0, k)
}

/// The induction claim for Jensen's inequality at bound k, over every pair of
/// sequences.
define jensen_claim(f: Real -> Real, a: Real, b: Real, k: Nat) -> Bool {
    forall(w0: Nat -> Real, x0: Nat -> Real) {
        jensen_claim_at(f, a, b, k, w0, x0)
    }
}

/// Applying the induction claim at one pair of sequences.
theorem jensen_claim_apply(f: Real -> Real, a: Real, b: Real, k: Nat, w0: Nat -> Real, x0: Nat -> Real) {
    jensen_claim(f, a, b, k) implies jensen_claim_at(f, a, b, k, w0, x0)
} by {
    if jensen_claim(f, a, b, k) {
        jensen_claim(f, a, b, k) = forall(w1: Nat -> Real, x1: Nat -> Real) {
            jensen_claim_at(f, a, b, k, w1, x1)
        }
        jensen_claim_at(f, a, b, k, w0, x0)
    }
}

/// Jensen's inequality for a convex function on an interval: for nonnegative
/// weights w(0), ..., w(n-1) summing to one and points x(0), ..., x(n-1) in
/// the interval [a, b],
/// f(sum_{i<n} w(i) * x(i)) <= sum_{i<n} w(i) * f(x(i)).
theorem jensen_convex_inequality(f: Real -> Real, a: Real, b: Real, w: Nat -> Real, x: Nat -> Real, n: Nat) {
    convex_on(f, a, b) and (forall(i: Nat) { Real.0 <= w(i) }) and partial(w, n) = Real.1 and
    jensen_points_in_interval(a, b, x, n)
    implies f(weighted_point_sum(w, x, n)) <= weighted_value_sum(f, w, x, n)
} by {
    if convex_on(f, a, b) and (forall(i: Nat) { Real.0 <= w(i) }) and partial(w, n) = Real.1 and
       jensen_points_in_interval(a, b, x, n) {
        forall(w0: Nat -> Real, x0: Nat -> Real) {
            if (forall(i: Nat) { Real.0 <= w0(i) }) and partial(w0, Nat.0) = Real.1 and
               jensen_points_in_interval(a, b, x0, Nat.0) {
                partial_zero(w0)
                partial(w0, Nat.0) = Real.0
                Real.0 = Real.1
                zero_is_different_than_one
                Real.0 != Real.1
                false
            }
        }
        forall(w0: Nat -> Real, x0: Nat -> Real) {
            jensen_claim_at(f, a, b, Nat.0, w0, x0) = ((forall(i: Nat) { Real.0 <= w0(i) }) and partial(w0, Nat.0) = Real.1 and jensen_points_in_interval(a, b, x0, Nat.0) implies (a <= weighted_point_sum(w0, x0, Nat.0) and weighted_point_sum(w0, x0, Nat.0) <= b) and f(weighted_point_sum(w0, x0, Nat.0)) <= weighted_value_sum(f, w0, x0, Nat.0))
            jensen_claim_at(f, a, b, Nat.0, w0, x0)
        }
        jensen_claim(f, a, b, Nat.0) = forall(w0: Nat -> Real, x0: Nat -> Real) {
            jensen_claim_at(f, a, b, Nat.0, w0, x0)
        }
        jensen_claim(f, a, b, Nat.0)

        forall(k: Nat) {
            if jensen_claim(f, a, b, k) {
                forall(w0: Nat -> Real, x0: Nat -> Real) {
                    if (forall(i: Nat) { Real.0 <= w0(i) }) and partial(w0, k.suc) = Real.1 and
                       jensen_points_in_interval(a, b, x0, k.suc) {
                        partial_split_last(w0, k)
                        partial(w0, k.suc) = partial(w0, k) + w0(k)
                        if w0(k) = Real.1 {
                            jensen_weights_split(w0, k)
                            partial(w0, k) = Real.1 - w0(k)
                            partial(w0, k) = Real.1 - Real.1
                            Real.1 - Real.1 = Real.1 + -Real.1
                            add_neg_eq_zero(Real.1)
                            Real.1 + -Real.1 = Real.0
                            Real.1 - Real.1 = Real.0
                            partial(w0, k) = Real.0
                            forall(i: Nat) {
                                if i < k {
                                    jensen_weight_zero_of_partial_zero(w0, k, i)
                                    w0(i) = Real.0
                                    pointwise_product(w0, x0, i) = w0(i) * x0(i)
                                    w0(i) * x0(i) = Real.0 * x0(i)
                                    mul_zero_left(x0(i))
                                    Real.0 * x0(i) = Real.0
                                    pointwise_product(w0, x0, i) = Real.0
                                }
                            }
                            partial_all_zeros(pointwise_product(w0, x0), k)
                            partial(pointwise_product(w0, x0), k) = Real.0
                            forall(i: Nat) {
                                if i < k {
                                    jensen_weight_zero_of_partial_zero(w0, k, i)
                                    w0(i) = Real.0
                                    pointwise_product(w0, pointwise_apply(f, x0), i) =
                                        w0(i) * pointwise_apply(f, x0, i)
                                    w0(i) * pointwise_apply(f, x0, i) = Real.0 * pointwise_apply(f, x0, i)
                                    mul_zero_left(pointwise_apply(f, x0, i))
                                    Real.0 * pointwise_apply(f, x0, i) = Real.0
                                    pointwise_product(w0, pointwise_apply(f, x0), i) = Real.0
                                }
                            }
                            partial_all_zeros(pointwise_product(w0, pointwise_apply(f, x0)), k)
                            partial(pointwise_product(w0, pointwise_apply(f, x0)), k) = Real.0
                            partial_split_last(pointwise_product(w0, x0), k)
                            partial(pointwise_product(w0, x0), k.suc) =
                                partial(pointwise_product(w0, x0), k) + pointwise_product(w0, x0, k)
                            pointwise_product(w0, x0, k) = w0(k) * x0(k)
                            w0(k) = Real.1
                            pointwise_product(w0, x0, k) = Real.1 * x0(k)
                            mul_one_left(x0(k))
                            Real.1 * x0(k) = x0(k)
                            pointwise_product(w0, x0, k) = x0(k)
                            partial(pointwise_product(w0, x0), k.suc) = Real.0 + x0(k)
                            add_zero_left(x0(k))
                            Real.0 + x0(k) = x0(k)
                            weighted_point_sum(w0, x0, k.suc) = partial(pointwise_product(w0, x0), k.suc)
                            weighted_point_sum(w0, x0, k.suc) = x0(k)
                            partial_split_last(pointwise_product(w0, pointwise_apply(f, x0)), k)
                            partial(pointwise_product(w0, pointwise_apply(f, x0)), k.suc) =
                                partial(pointwise_product(w0, pointwise_apply(f, x0)), k) +
                                pointwise_product(w0, pointwise_apply(f, x0), k)
                            pointwise_product(w0, pointwise_apply(f, x0), k) =
                                w0(k) * pointwise_apply(f, x0, k)
                            pointwise_apply(f, x0, k) = f(x0(k))
                            pointwise_product(w0, pointwise_apply(f, x0), k) = Real.1 * f(x0(k))
                            mul_one_left(f(x0(k)))
                            Real.1 * f(x0(k)) = f(x0(k))
                            pointwise_product(w0, pointwise_apply(f, x0), k) = f(x0(k))
                            partial(pointwise_product(w0, pointwise_apply(f, x0)), k.suc) =
                                Real.0 + f(x0(k))
                            add_zero_left(f(x0(k)))
                            Real.0 + f(x0(k)) = f(x0(k))
                            weighted_value_sum(f, w0, x0, k.suc) =
                                partial(pointwise_product(w0, pointwise_apply(f, x0)), k.suc)
                            weighted_value_sum(f, w0, x0, k.suc) = f(x0(k))
                            lte_refl(f(x0(k)))
                            f(x0(k)) <= f(x0(k))
                            f(weighted_point_sum(w0, x0, k.suc)) = f(x0(k))
                            f(weighted_point_sum(w0, x0, k.suc)) <= weighted_value_sum(f, w0, x0, k.suc)
                            lt_suc(k)
                            k < k.suc
                            jensen_points_in_interval_apply(a, b, x0, k.suc, k)
                            a <= x0(k) and x0(k) <= b
                            a <= x0(k)
                            weighted_point_sum(w0, x0, k.suc) = x0(k)
                            x0(k) = weighted_point_sum(w0, x0, k.suc)
                            a <= weighted_point_sum(w0, x0, k.suc)
                            x0(k) <= b
                            weighted_point_sum(w0, x0, k.suc) <= b
                            a <= weighted_point_sum(w0, x0, k.suc) and
                                weighted_point_sum(w0, x0, k.suc) <= b
                            (a <= weighted_point_sum(w0, x0, k.suc) and
                                weighted_point_sum(w0, x0, k.suc) <= b) and
                                f(weighted_point_sum(w0, x0, k.suc)) <= weighted_value_sum(f, w0, x0, k.suc)
                        } else {
                            w0(k) != Real.1
                            jensen_weights_split(w0, k)
                            partial(w0, k) = Real.1 - w0(k)
                            is_lower_bound(w0, Real.0) = forall(i: Nat) { Real.0 <= w0(i) }
                            is_lower_bound(w0, Real.0)
                            partial_nonneg(w0, k)
                            partial(w0, k) >= Real.0
                            Real.0 <= partial(w0, k)
                            Real.1 - w0(k) = Real.1 + -w0(k)
                            Real.0 <= Real.1 + -w0(k)
                            lte_add_right(Real.0, Real.1 + -w0(k), w0(k))
                            Real.0 + w0(k) <= (Real.1 + -w0(k)) + w0(k)
                            add_zero_left(w0(k))
                            Real.0 + w0(k) = w0(k)
                            w0(k) <= (Real.1 + -w0(k)) + w0(k)
                            add_assoc(Real.1, -w0(k), w0(k))
                            (Real.1 + -w0(k)) + w0(k) = Real.1 + (-w0(k) + w0(k))
                            add_comm(-w0(k), w0(k))
                            -w0(k) + w0(k) = w0(k) + -w0(k)
                            add_neg_eq_zero(w0(k))
                            w0(k) + -w0(k) = Real.0
                            Real.1 + (-w0(k) + w0(k)) = Real.1 + Real.0
                            add_zero_right(Real.1)
                            Real.1 + Real.0 = Real.1
                            (Real.1 + -w0(k)) + w0(k) = Real.1
                            w0(k) <= Real.1
                            lt_of_lte_of_ne[Real](w0(k), Real.1)
                            w0(k) < Real.1
                            lt_add_right(w0(k), Real.1, -w0(k))
                            w0(k) + -w0(k) < Real.1 + -w0(k)
                            add_neg_eq_zero(w0(k))
                            w0(k) + -w0(k) = Real.0
                            Real.0 < Real.1 + -w0(k)
                            Real.1 - w0(k) = Real.1 + -w0(k)
                            Real.0 < Real.1 - w0(k)
                            lt_imp_ne(Real.0, Real.1 - w0(k))
                            Real.1 - w0(k) != Real.0
                            forall(i: Nat) {
                                scaled_weight_nonneg(Real.1 - w0(k), w0, i)
                                Real.0 <= scaled_weight(Real.1 - w0(k), w0, i)
                            }
                            partial_scaled_weight(Real.1 - w0(k), w0, k)
                            partial(scaled_weight(Real.1 - w0(k), w0), k) =
                                (Real.1 - w0(k)).inverse * partial(w0, k)
                            real_mul_comm((Real.1 - w0(k)).inverse, Real.1 - w0(k))
                            (Real.1 - w0(k)).inverse * (Real.1 - w0(k)) =
                                (Real.1 - w0(k)) * (Real.1 - w0(k)).inverse
                            mul_inverse(Real.1 - w0(k))
                            (Real.1 - w0(k)) * (Real.1 - w0(k)).inverse = Real.1
                            (Real.1 - w0(k)).inverse * (Real.1 - w0(k)) = Real.1
                            partial(scaled_weight(Real.1 - w0(k), w0), k) =
                                (Real.1 - w0(k)).inverse * (Real.1 - w0(k))
                            partial(scaled_weight(Real.1 - w0(k), w0), k) = Real.1
                            partial_weighted_scaled(Real.1 - w0(k), w0, x0, k)
                            partial(pointwise_product(w0, x0), k) = (Real.1 - w0(k)) * partial(pointwise_product(scaled_weight(Real.1 - w0(k), w0), x0), k)
                            weighted_point_sum(w0, x0, k) = partial(pointwise_product(w0, x0), k)
                            weighted_point_sum(scaled_weight(Real.1 - w0(k), w0), x0, k) = partial(pointwise_product(scaled_weight(Real.1 - w0(k), w0), x0), k)
                            weighted_point_sum(w0, x0, k) = (Real.1 - w0(k)) * weighted_point_sum(scaled_weight(Real.1 - w0(k), w0), x0, k)
                            partial_weighted_scaled(Real.1 - w0(k), w0, pointwise_apply(f, x0), k)
                            partial(pointwise_product(w0, pointwise_apply(f, x0)), k) = (Real.1 - w0(k)) * partial(pointwise_product(scaled_weight(Real.1 - w0(k), w0), pointwise_apply(f, x0)), k)
                            weighted_value_sum(f, w0, x0, k) = partial(pointwise_product(w0, pointwise_apply(f, x0)), k)
                            weighted_value_sum(f, scaled_weight(Real.1 - w0(k), w0), x0, k) = partial(pointwise_product(scaled_weight(Real.1 - w0(k), w0), pointwise_apply(f, x0)), k)
                            weighted_value_sum(f, w0, x0, k) = (Real.1 - w0(k)) * weighted_value_sum(f, scaled_weight(Real.1 - w0(k), w0), x0, k)
                            jensen_points_in_interval_restrict(a, b, x0, k)
                            jensen_points_in_interval(a, b, x0, k)
                            jensen_claim_apply(f, a, b, k, scaled_weight(Real.1 - w0(k), w0), x0)
                            jensen_claim_at(f, a, b, k, scaled_weight(Real.1 - w0(k), w0), x0)
                            jensen_claim_at(f, a, b, k, scaled_weight(Real.1 - w0(k), w0), x0) = ((forall(i: Nat) { Real.0 <= scaled_weight(Real.1 - w0(k), w0, i) }) and partial(scaled_weight(Real.1 - w0(k), w0), k) = Real.1 and jensen_points_in_interval(a, b, x0, k) implies (a <= weighted_point_sum(scaled_weight(Real.1 - w0(k), w0), x0, k) and weighted_point_sum(scaled_weight(Real.1 - w0(k), w0), x0, k) <= b) and f(weighted_point_sum(scaled_weight(Real.1 - w0(k), w0), x0, k)) <= weighted_value_sum(f, scaled_weight(Real.1 - w0(k), w0), x0, k))
                            (forall(i: Nat) { Real.0 <= scaled_weight(Real.1 - w0(k), w0, i) }) and partial(scaled_weight(Real.1 - w0(k), w0), k) = Real.1 and jensen_points_in_interval(a, b, x0, k) implies (a <= weighted_point_sum(scaled_weight(Real.1 - w0(k), w0), x0, k) and weighted_point_sum(scaled_weight(Real.1 - w0(k), w0), x0, k) <= b) and f(weighted_point_sum(scaled_weight(Real.1 - w0(k), w0), x0, k)) <= weighted_value_sum(f, scaled_weight(Real.1 - w0(k), w0), x0, k)
                            (forall(i: Nat) { Real.0 <= scaled_weight(Real.1 - w0(k), w0, i) }) and partial(scaled_weight(Real.1 - w0(k), w0), k) = Real.1 and jensen_points_in_interval(a, b, x0, k)
                            (a <= weighted_point_sum(scaled_weight(Real.1 - w0(k), w0), x0, k) and weighted_point_sum(scaled_weight(Real.1 - w0(k), w0), x0, k) <= b) and f(weighted_point_sum(scaled_weight(Real.1 - w0(k), w0), x0, k)) <= weighted_value_sum(f, scaled_weight(Real.1 - w0(k), w0), x0, k)
                            a <= weighted_point_sum(scaled_weight(Real.1 - w0(k), w0), x0, k)
                            weighted_point_sum(scaled_weight(Real.1 - w0(k), w0), x0, k) <= b
                            f(weighted_point_sum(scaled_weight(Real.1 - w0(k), w0), x0, k)) <= weighted_value_sum(f, scaled_weight(Real.1 - w0(k), w0), x0, k)
                            lt_suc(k)
                            k < k.suc
                            jensen_points_in_interval_apply(a, b, x0, k.suc, k)
                            a <= x0(k) and x0(k) <= b
                            convex_weighted_inequality(f, a, b, x0(k), weighted_point_sum(scaled_weight(Real.1 - w0(k), w0), x0, k), w0(k))
                            f(w0(k) * x0(k) + (Real.1 - w0(k)) * weighted_point_sum(scaled_weight(Real.1 - w0(k), w0), x0, k)) <= w0(k) * f(x0(k)) + (Real.1 - w0(k)) * f(weighted_point_sum(scaled_weight(Real.1 - w0(k), w0), x0, k))
                            partial_split_last(pointwise_product(w0, x0), k)
                            partial(pointwise_product(w0, x0), k.suc) =
                                partial(pointwise_product(w0, x0), k) + pointwise_product(w0, x0, k)
                            pointwise_product(w0, x0, k) = w0(k) * x0(k)
                            weighted_point_sum(w0, x0, k.suc) = partial(pointwise_product(w0, x0), k.suc)
                            weighted_point_sum(w0, x0, k.suc) =
                                weighted_point_sum(w0, x0, k) + w0(k) * x0(k)
                            weighted_point_sum(w0, x0, k.suc) = (Real.1 - w0(k)) * weighted_point_sum(scaled_weight(Real.1 - w0(k), w0), x0, k) + w0(k) * x0(k)
                            add_comm(w0(k) * x0(k), (Real.1 - w0(k)) * weighted_point_sum(scaled_weight(Real.1 - w0(k), w0), x0, k))
                            w0(k) * x0(k) + (Real.1 - w0(k)) * weighted_point_sum(scaled_weight(Real.1 - w0(k), w0), x0, k) = (Real.1 - w0(k)) * weighted_point_sum(scaled_weight(Real.1 - w0(k), w0), x0, k) + w0(k) * x0(k)
                            f(weighted_point_sum(w0, x0, k.suc)) = f(w0(k) * x0(k) + (Real.1 - w0(k)) * weighted_point_sum(scaled_weight(Real.1 - w0(k), w0), x0, k))
                            f(weighted_point_sum(w0, x0, k.suc)) <= w0(k) * f(x0(k)) + (Real.1 - w0(k)) * f(weighted_point_sum(scaled_weight(Real.1 - w0(k), w0), x0, k))
                            mul_le_mul_of_nonneg_right[Real](f(weighted_point_sum(scaled_weight(Real.1 - w0(k), w0), x0, k)), weighted_value_sum(f, scaled_weight(Real.1 - w0(k), w0), x0, k), Real.1 - w0(k))
                            f(weighted_point_sum(scaled_weight(Real.1 - w0(k), w0), x0, k)) * (Real.1 - w0(k)) <= weighted_value_sum(f, scaled_weight(Real.1 - w0(k), w0), x0, k) * (Real.1 - w0(k))
                            real_mul_comm(f(weighted_point_sum(scaled_weight(Real.1 - w0(k), w0), x0, k)), Real.1 - w0(k))
                            f(weighted_point_sum(scaled_weight(Real.1 - w0(k), w0), x0, k)) * (Real.1 - w0(k)) = (Real.1 - w0(k)) * f(weighted_point_sum(scaled_weight(Real.1 - w0(k), w0), x0, k))
                            real_mul_comm(weighted_value_sum(f, scaled_weight(Real.1 - w0(k), w0), x0, k), Real.1 - w0(k))
                            weighted_value_sum(f, scaled_weight(Real.1 - w0(k), w0), x0, k) * (Real.1 - w0(k)) = (Real.1 - w0(k)) * weighted_value_sum(f, scaled_weight(Real.1 - w0(k), w0), x0, k)
                            (Real.1 - w0(k)) * f(weighted_point_sum(scaled_weight(Real.1 - w0(k), w0), x0, k)) <= (Real.1 - w0(k)) * weighted_value_sum(f, scaled_weight(Real.1 - w0(k), w0), x0, k)
                            (Real.1 - w0(k)) * weighted_value_sum(f, scaled_weight(Real.1 - w0(k), w0), x0, k) = weighted_value_sum(f, w0, x0, k)
                            (Real.1 - w0(k)) * f(weighted_point_sum(scaled_weight(Real.1 - w0(k), w0), x0, k)) <= weighted_value_sum(f, w0, x0, k)
                            lte_add_left((Real.1 - w0(k)) * f(weighted_point_sum(scaled_weight(Real.1 - w0(k), w0), x0, k)), weighted_value_sum(f, w0, x0, k), w0(k) * f(x0(k)))
                            w0(k) * f(x0(k)) + (Real.1 - w0(k)) * f(weighted_point_sum(scaled_weight(Real.1 - w0(k), w0), x0, k)) <= w0(k) * f(x0(k)) + weighted_value_sum(f, w0, x0, k)
                            add_comm(w0(k) * f(x0(k)), weighted_value_sum(f, w0, x0, k))
                            w0(k) * f(x0(k)) + weighted_value_sum(f, w0, x0, k) =
                                weighted_value_sum(f, w0, x0, k) + w0(k) * f(x0(k))
                            partial_split_last(pointwise_product(w0, pointwise_apply(f, x0)), k)
                            partial(pointwise_product(w0, pointwise_apply(f, x0)), k.suc) =
                                partial(pointwise_product(w0, pointwise_apply(f, x0)), k) +
                                pointwise_product(w0, pointwise_apply(f, x0), k)
                            pointwise_product(w0, pointwise_apply(f, x0), k) =
                                w0(k) * pointwise_apply(f, x0, k)
                            pointwise_apply(f, x0, k) = f(x0(k))
                            pointwise_product(w0, pointwise_apply(f, x0), k) = w0(k) * f(x0(k))
                            weighted_value_sum(f, w0, x0, k.suc) =
                                partial(pointwise_product(w0, pointwise_apply(f, x0)), k.suc)
                            weighted_value_sum(f, w0, x0, k.suc) =
                                weighted_value_sum(f, w0, x0, k) + w0(k) * f(x0(k))
                            w0(k) * f(x0(k)) + weighted_value_sum(f, w0, x0, k) =
                                weighted_value_sum(f, w0, x0, k.suc)
                            lte_trans(f(weighted_point_sum(w0, x0, k.suc)), w0(k) * f(x0(k)) + (Real.1 - w0(k)) * f(weighted_point_sum(scaled_weight(Real.1 - w0(k), w0), x0, k)), weighted_value_sum(f, w0, x0, k.suc))
                            f(weighted_point_sum(w0, x0, k.suc)) <= weighted_value_sum(f, w0, x0, k.suc)
                            convex_combination_in_closed_interval(a, b, weighted_point_sum(scaled_weight(Real.1 - w0(k), w0), x0, k), x0(k), w0(k))
                            a <= (Real.1 - w0(k)) * weighted_point_sum(scaled_weight(Real.1 - w0(k), w0), x0, k) + w0(k) * x0(k) and (Real.1 - w0(k)) * weighted_point_sum(scaled_weight(Real.1 - w0(k), w0), x0, k) + w0(k) * x0(k) <= b
                            (Real.1 - w0(k)) * weighted_point_sum(scaled_weight(Real.1 - w0(k), w0), x0, k) + w0(k) * x0(k) = weighted_point_sum(w0, x0, k.suc)
                            a <= (Real.1 - w0(k)) * weighted_point_sum(scaled_weight(Real.1 - w0(k), w0), x0, k) + w0(k) * x0(k)
                            a <= weighted_point_sum(w0, x0, k.suc)
                            (Real.1 - w0(k)) * weighted_point_sum(scaled_weight(Real.1 - w0(k), w0), x0, k) + w0(k) * x0(k) <= b
                            weighted_point_sum(w0, x0, k.suc) <= b
                            a <= weighted_point_sum(w0, x0, k.suc) and weighted_point_sum(w0, x0, k.suc) <= b
                            (a <= weighted_point_sum(w0, x0, k.suc) and weighted_point_sum(w0, x0, k.suc) <= b) and f(weighted_point_sum(w0, x0, k.suc)) <= weighted_value_sum(f, w0, x0, k.suc)
                        }
                    }
                }
                forall(w0: Nat -> Real, x0: Nat -> Real) {
                    jensen_claim_at(f, a, b, k.suc, w0, x0) = ((forall(i: Nat) { Real.0 <= w0(i) }) and partial(w0, k.suc) = Real.1 and jensen_points_in_interval(a, b, x0, k.suc) implies (a <= weighted_point_sum(w0, x0, k.suc) and weighted_point_sum(w0, x0, k.suc) <= b) and f(weighted_point_sum(w0, x0, k.suc)) <= weighted_value_sum(f, w0, x0, k.suc))
                    jensen_claim_at(f, a, b, k.suc, w0, x0)
                }
                jensen_claim(f, a, b, k.suc) = forall(w0: Nat -> Real, x0: Nat -> Real) {
                    jensen_claim_at(f, a, b, k.suc, w0, x0)
                }
                jensen_claim(f, a, b, k.suc)
            }
        }

        jensen_claim(f, a, b, Nat.0) and forall(k: Nat) { jensen_claim(f, a, b, k) implies jensen_claim(f, a, b, k.suc) }
        alt_induction(jensen_claim(f, a, b))
        forall(k: Nat) { jensen_claim(f, a, b, k) }
        jensen_claim(f, a, b, n)
        jensen_claim(f, a, b, n) = forall(w0: Nat -> Real, x0: Nat -> Real) {
            jensen_claim_at(f, a, b, n, w0, x0)
        }
        jensen_claim_at(f, a, b, n, w, x)
        jensen_claim_at(f, a, b, n, w, x) = ((forall(i: Nat) { Real.0 <= w(i) }) and partial(w, n) = Real.1 and jensen_points_in_interval(a, b, x, n) implies (a <= weighted_point_sum(w, x, n) and weighted_point_sum(w, x, n) <= b) and f(weighted_point_sum(w, x, n)) <= weighted_value_sum(f, w, x, n))
        (forall(i: Nat) { Real.0 <= w(i) }) and partial(w, n) = Real.1 and jensen_points_in_interval(a, b, x, n) implies (a <= weighted_point_sum(w, x, n) and weighted_point_sum(w, x, n) <= b) and f(weighted_point_sum(w, x, n)) <= weighted_value_sum(f, w, x, n)
        (forall(i: Nat) { Real.0 <= w(i) }) and partial(w, n) = Real.1 and jensen_points_in_interval(a, b, x, n)
        f(weighted_point_sum(w, x, n)) <= weighted_value_sum(f, w, x, n)
    }
}
