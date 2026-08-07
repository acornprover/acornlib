from data.basic.set import Set, intersection_contains_eq, intersection_contains_intro
from real.real_field import Real
from real.topology import disconnecting_triple, is_connected_real_set

/// A connected real set contains every point between two of its points.
theorem connected_real_set_contains_between(s: Set[Real], x: Real, y: Real, z: Real) {
    is_connected_real_set(s) and s.contains(x) and s.contains(y) and x < y and x <= z and z <= y
    implies s.contains(z)
} by {
    if is_connected_real_set(s) and s.contains(x) and s.contains(y) and x < y and x <= z and z <= y {
        if not s.contains(z) {
            disconnecting_triple(s, x, y, z)
            is_connected_real_set(s) = forall(a: Real, b: Real, c: Real) {
                not disconnecting_triple(s, a, b, c)
            }
            not disconnecting_triple(s, x, y, z)
            false
        }
        s.contains(z)
    }
}

/// A point between two points of an intersection lies in the left set.
theorem connected_intersection_between_left(s: Set[Real], t: Set[Real], x: Real, y: Real, z: Real) {
    is_connected_real_set(s) and s.intersection(t).contains(x) and s.intersection(t).contains(y) and
    x < y and x <= z and z <= y implies s.contains(z)
} by {
    if is_connected_real_set(s) and s.intersection(t).contains(x) and s.intersection(t).contains(y) and
       x < y and x <= z and z <= y {
        intersection_contains_eq(s, t, x)
        intersection_contains_eq(s, t, y)
        s.contains(x)
        s.contains(y)
        connected_real_set_contains_between(s, x, y, z)
        s.contains(z)
    }
}

/// A point between two points of an intersection lies in the right set.
theorem connected_intersection_between_right(s: Set[Real], t: Set[Real], x: Real, y: Real, z: Real) {
    is_connected_real_set(t) and s.intersection(t).contains(x) and s.intersection(t).contains(y) and
    x < y and x <= z and z <= y implies t.contains(z)
} by {
    if is_connected_real_set(t) and s.intersection(t).contains(x) and s.intersection(t).contains(y) and
       x < y and x <= z and z <= y {
        intersection_contains_eq(s, t, x)
        intersection_contains_eq(s, t, y)
        t.contains(x)
        t.contains(y)
        connected_real_set_contains_between(t, x, y, z)
        t.contains(z)
    }
}

/// The intersection of two connected real sets is connected.
theorem intersection_of_connected_real_sets_is_connected(s: Set[Real], t: Set[Real]) {
    is_connected_real_set(s) and is_connected_real_set(t) implies is_connected_real_set(s.intersection(t))
} by {
    if is_connected_real_set(s) and is_connected_real_set(t) {
        forall(x: Real, y: Real, z: Real) {
            if disconnecting_triple(s.intersection(t), x, y, z) {
                disconnecting_triple(s.intersection(t), x, y, z) =
                    (s.intersection(t).contains(x) and s.intersection(t).contains(y) and x < y and
                     x <= z and z <= y and not s.intersection(t).contains(z))
                s.intersection(t).contains(x)
                s.intersection(t).contains(y)
                x < y
                x <= z
                z <= y
                connected_intersection_between_left(s, t, x, y, z)
                connected_intersection_between_right(s, t, x, y, z)
                s.contains(z)
                t.contains(z)
                intersection_contains_intro(s, t, z)
                s.intersection(t).contains(z)
                not s.intersection(t).contains(z)
                false
            }
        }
        is_connected_real_set(s.intersection(t))
    }
}

/// A singleton real set is connected.
theorem singleton_real_set_is_connected(a: Real) {
    is_connected_real_set(Set[Real].singleton(a))
} by {
    forall(x: Real, y: Real, z: Real) {
        if disconnecting_triple(Set[Real].singleton(a), x, y, z) {
            disconnecting_triple(Set[Real].singleton(a), x, y, z) =
                (Set[Real].singleton(a).contains(x) and Set[Real].singleton(a).contains(y) and
                 x < y and x <= z and z <= y and not Set[Real].singleton(a).contains(z))
            Set[Real].singleton(a).contains(x)
            Set[Real].singleton(a).contains(y)
            x = a
            y = a
            x = y
            x < y
            false
        }
    }
}

/// Connectedness is preserved by equality of real sets.
theorem connected_real_set_of_eq(s: Set[Real], t: Set[Real]) {
    s = t and is_connected_real_set(s) implies is_connected_real_set(t)
} by {
    if s = t and is_connected_real_set(s) {
        is_connected_real_set(t)
    }
}

/// Connectedness is preserved by equality of real sets in the reverse direction.
theorem connected_real_set_of_eq_rev(s: Set[Real], t: Set[Real]) {
    s = t and is_connected_real_set(t) implies is_connected_real_set(s)
} by {
    if s = t and is_connected_real_set(t) {
        is_connected_real_set(s)
    }
}
