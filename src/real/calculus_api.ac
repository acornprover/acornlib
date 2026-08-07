/// Public calculus API for real derivatives.

from data.basic.function_algebra import pointwise_add, pointwise_neg, pointwise_mul
from data.basic.functions import identity_fn, compose
from real.real_base import Real
from real.derivative_basic import has_derivative_at, differentiable_at,
    has_derivative_at_unique, constant_has_derivative_at,
    identity_has_derivative_at, constant_differentiable_at,
    identity_differentiable_at
from real.derivative_rules import derivative_pointwise_neg,
    derivative_pointwise_add, derivative_pointwise_sub,
    differentiable_pointwise_neg, differentiable_pointwise_add,
    differentiable_pointwise_sub
from real.derivative_linear import derivative_pointwise_const_mul,
    derivative_pointwise_mul_const, derivative_pointwise_linear_combination,
    affine_identity_has_derivative_at, affine_identity_differentiable_at,
    differentiable_pointwise_const_mul, differentiable_pointwise_mul_const,
    differentiable_pointwise_linear_combination
from real.derivative_product import derivative_pointwise_mul,
    derivative_pointwise_square, differentiable_pointwise_mul,
    differentiable_pointwise_square
from real.derivative_chain import derivative_compose, differentiable_compose
from real.derivative_continuity import derivative_continuous_at,
    differentiable_continuous_at
from real.continuity_base import continuous_at

/// A function `df` is the pointwise derivative of `f` on all real inputs.
define is_derivative_fn(f: Real -> Real, df: Real -> Real) -> Bool {
    forall(x: Real) {
        has_derivative_at(f, x, df(x))
    }
}

/// A real function is differentiable at every point.
define differentiable_everywhere(f: Real -> Real) -> Bool {
    forall(x: Real) {
        differentiable_at(f, x)
    }
}

/// Instantiating differentiability everywhere at a single point.
theorem differentiable_everywhere_at(f: Real -> Real, x: Real) {
    differentiable_everywhere(f) implies differentiable_at(f, x)
} by {
    if differentiable_everywhere(f) {
        differentiable_everywhere(f) = forall(y: Real) {
            differentiable_at(f, y)
        }
        differentiable_at(f, x)
    }
}

/// Unfolding lemma for global derivative functions.
theorem is_derivative_fn_iff(f: Real -> Real, df: Real -> Real) {
    is_derivative_fn(f, df) = forall(x: Real) {
        has_derivative_at(f, x, df(x))
    }
} by {
    is_derivative_fn(f, df) = forall(x: Real) {
        has_derivative_at(f, x, df(x))
    }
}

/// A global derivative function supplies a derivative value at each point.
theorem is_derivative_fn_at(f: Real -> Real, df: Real -> Real, x: Real) {
    is_derivative_fn(f, df) implies has_derivative_at(f, x, df(x))
} by {
    if is_derivative_fn(f, df) {
        is_derivative_fn_iff(f, df)
        forall(y: Real) {
            has_derivative_at(f, y, df(y))
        }
        has_derivative_at(f, x, df(x))
    }
}

/// A global derivative function makes the function differentiable everywhere.
theorem is_derivative_fn_imp_differentiable_everywhere(f: Real -> Real, df: Real -> Real) {
    is_derivative_fn(f, df) implies differentiable_everywhere(f)
} by {
    if is_derivative_fn(f, df) {
        forall(x: Real) {
            is_derivative_fn_at(f, df, x)
            exists(d: Real) {
                has_derivative_at(f, x, d)
            }
            differentiable_at(f, x)
        }
    }
}

/// A global derivative function makes the function differentiable at any specified point.
theorem is_derivative_fn_imp_differentiable_at(f: Real -> Real, df: Real -> Real, x: Real) {
    is_derivative_fn(f, df) implies differentiable_at(f, x)
} by {
    if is_derivative_fn(f, df) {
        is_derivative_fn_imp_differentiable_everywhere(f, df)
        differentiable_everywhere_at(f, x)
        differentiable_at(f, x)
    }
}

/// The value of a global derivative function is unique at each point.
theorem derivative_fn_value_unique(
    f: Real -> Real, df: Real -> Real, x: Real, d: Real
) {
    is_derivative_fn(f, df) and has_derivative_at(f, x, d) implies df(x) = d
} by {
    if is_derivative_fn(f, df) and has_derivative_at(f, x, d) {
        is_derivative_fn_at(f, df, x)
        has_derivative_at_unique(f, x, df(x), d)
        df(x) = d
    }
}


/// Constant real functions have the constant-zero function as global derivative.
theorem derivative_fn_constant(c: Real) {
    is_derivative_fn(constant[Real, Real](c), constant[Real, Real](Real.0))
} by {
    forall(x: Real) {
        constant_has_derivative_at(c, x)
        constant[Real, Real](Real.0, x) = Real.0
        has_derivative_at(constant[Real, Real](c), x, constant[Real, Real](Real.0, x))
    }
}

/// The identity function has the constant-one function as global derivative.
theorem derivative_fn_identity {
    is_derivative_fn(identity_fn[Real], constant[Real, Real](Real.1))
} by {
    forall(x: Real) {
        identity_has_derivative_at(x)
        constant[Real, Real](Real.1, x) = Real.1
        has_derivative_at(identity_fn[Real], x, constant[Real, Real](Real.1, x))
    }
}

/// Constant real functions are differentiable everywhere.
theorem differentiable_everywhere_constant(c: Real) {
    differentiable_everywhere(constant[Real, Real](c))
} by {
    forall(x: Real) {
        constant_differentiable_at(c, x)
    }
}

/// The identity function is differentiable everywhere.
theorem differentiable_everywhere_identity {
    differentiable_everywhere(identity_fn[Real])
} by {
    forall(x: Real) {
        identity_differentiable_at(x)
    }
}

/// Pointwise negation transports global derivative functions.
theorem derivative_fn_neg(f: Real -> Real, df: Real -> Real) {
    is_derivative_fn(f, df) implies is_derivative_fn(pointwise_neg(f), pointwise_neg(df))
} by {
    if is_derivative_fn(f, df) {
        forall(x: Real) {
            is_derivative_fn_at(f, df, x)
            derivative_pointwise_neg(f, x, df(x))
            has_derivative_at(pointwise_neg(f), x, -df(x))
            pointwise_neg(df, x) = -df(x)
            has_derivative_at(pointwise_neg(f), x, pointwise_neg(df, x))
        }
    }
}

/// Pointwise addition transports global derivative functions.
theorem derivative_fn_add(f: Real -> Real, g: Real -> Real, df: Real -> Real, dg: Real -> Real) {
    is_derivative_fn(f, df) and is_derivative_fn(g, dg)
        implies is_derivative_fn(pointwise_add(f, g), pointwise_add(df, dg))
} by {
    if is_derivative_fn(f, df) and is_derivative_fn(g, dg) {
        forall(x: Real) {
            is_derivative_fn_at(f, df, x)
            is_derivative_fn_at(g, dg, x)
            derivative_pointwise_add(f, g, x, df(x), dg(x))
            has_derivative_at(pointwise_add(f, g), x, df(x) + dg(x))
            pointwise_add(df, dg, x) = df(x) + dg(x)
            has_derivative_at(pointwise_add(f, g), x, pointwise_add(df, dg, x))
        }
    }
}

/// Pointwise subtraction transports global derivative functions.
theorem derivative_fn_sub(f: Real -> Real, g: Real -> Real, df: Real -> Real, dg: Real -> Real) {
    is_derivative_fn(f, df) and is_derivative_fn(g, dg)
        implies is_derivative_fn(pointwise_add(f, pointwise_neg(g)), pointwise_add(df, pointwise_neg(dg)))
} by {
    if is_derivative_fn(f, df) and is_derivative_fn(g, dg) {
        forall(x: Real) {
            is_derivative_fn_at(f, df, x)
            is_derivative_fn_at(g, dg, x)
            derivative_pointwise_sub(f, g, x, df(x), dg(x))
            has_derivative_at(pointwise_add(f, pointwise_neg(g)), x, df(x) + -dg(x))
            pointwise_neg(dg, x) = -dg(x)
            pointwise_add(df, pointwise_neg(dg), x) = df(x) + pointwise_neg(dg, x)
            pointwise_add(df, pointwise_neg(dg), x) = df(x) + -dg(x)
            has_derivative_at(pointwise_add(f, pointwise_neg(g)), x, pointwise_add(df, pointwise_neg(dg), x))
        }
    }
}

/// Left scalar multiplication transports global derivative functions.
theorem derivative_fn_const_mul(c: Real, f: Real -> Real, df: Real -> Real) {
    is_derivative_fn(f, df)
        implies is_derivative_fn(pointwise_mul(constant[Real, Real](c), f), pointwise_mul(constant[Real, Real](c), df))
} by {
    if is_derivative_fn(f, df) {
        forall(x: Real) {
            is_derivative_fn_at(f, df, x)
            derivative_pointwise_const_mul(c, f, x, df(x))
            has_derivative_at(pointwise_mul(constant[Real, Real](c), f), x, c * df(x))
            constant[Real, Real](c, x) = c
            pointwise_mul(constant[Real, Real](c), df, x) = constant[Real, Real](c, x) * df(x)
            pointwise_mul(constant[Real, Real](c), df, x) = c * df(x)
            has_derivative_at(pointwise_mul(constant[Real, Real](c), f), x,
                pointwise_mul(constant[Real, Real](c), df, x))
        }
    }
}

/// Right scalar multiplication transports global derivative functions.
theorem derivative_fn_mul_const(c: Real, f: Real -> Real, df: Real -> Real) {
    is_derivative_fn(f, df)
        implies is_derivative_fn(pointwise_mul(f, constant[Real, Real](c)), pointwise_mul(df, constant[Real, Real](c)))
} by {
    if is_derivative_fn(f, df) {
        forall(x: Real) {
            is_derivative_fn_at(f, df, x)
            derivative_pointwise_mul_const(c, f, x, df(x))
            has_derivative_at(pointwise_mul(f, constant[Real, Real](c)), x, df(x) * c)
            constant[Real, Real](c, x) = c
            pointwise_mul(df, constant[Real, Real](c), x) = df(x) * constant[Real, Real](c, x)
            pointwise_mul(df, constant[Real, Real](c), x) = df(x) * c
            has_derivative_at(pointwise_mul(f, constant[Real, Real](c)), x,
                pointwise_mul(df, constant[Real, Real](c), x))
        }
    }
}

/// Linear combinations transport global derivative functions.
theorem derivative_fn_linear_combination(
    a: Real, b: Real, f: Real -> Real, g: Real -> Real, df: Real -> Real, dg: Real -> Real
) {
    is_derivative_fn(f, df) and is_derivative_fn(g, dg) implies is_derivative_fn(
        pointwise_add(pointwise_mul(constant[Real, Real](a), f), pointwise_mul(constant[Real, Real](b), g)),
        pointwise_add(pointwise_mul(constant[Real, Real](a), df), pointwise_mul(constant[Real, Real](b), dg))
    )
} by {
    if is_derivative_fn(f, df) and is_derivative_fn(g, dg) {
        forall(x: Real) {
            is_derivative_fn_at(f, df, x)
            is_derivative_fn_at(g, dg, x)
            derivative_pointwise_linear_combination(a, b, f, g, x, df(x), dg(x))
            has_derivative_at(
                pointwise_add(pointwise_mul(constant[Real, Real](a), f), pointwise_mul(constant[Real, Real](b), g)),
                x,
                a * df(x) + b * dg(x)
            )
            constant[Real, Real](a, x) = a
            constant[Real, Real](b, x) = b
            pointwise_mul(constant[Real, Real](a), df, x) = a * df(x)
            pointwise_mul(constant[Real, Real](b), dg, x) = b * dg(x)
            pointwise_add(pointwise_mul(constant[Real, Real](a), df), pointwise_mul(constant[Real, Real](b), dg), x) =
                pointwise_mul(constant[Real, Real](a), df, x) + pointwise_mul(constant[Real, Real](b), dg, x)
            pointwise_add(pointwise_mul(constant[Real, Real](a), df), pointwise_mul(constant[Real, Real](b), dg), x) =
                a * df(x) + b * dg(x)
            has_derivative_at(
                pointwise_add(pointwise_mul(constant[Real, Real](a), f), pointwise_mul(constant[Real, Real](b), g)),
                x,
                pointwise_add(pointwise_mul(constant[Real, Real](a), df), pointwise_mul(constant[Real, Real](b), dg), x)
            )
        }
    }
}

/// Pointwise products transport global derivative functions by the product rule.
theorem derivative_fn_mul(f: Real -> Real, g: Real -> Real, df: Real -> Real, dg: Real -> Real) {
    is_derivative_fn(f, df) and is_derivative_fn(g, dg)
        implies is_derivative_fn(pointwise_mul(f, g), pointwise_add(pointwise_mul(f, dg), pointwise_mul(g, df)))
} by {
    if is_derivative_fn(f, df) and is_derivative_fn(g, dg) {
        forall(x: Real) {
            is_derivative_fn_at(f, df, x)
            is_derivative_fn_at(g, dg, x)
            derivative_pointwise_mul(f, g, x, df(x), dg(x))
            has_derivative_at(pointwise_mul(f, g), x, f(x) * dg(x) + g(x) * df(x))
            pointwise_mul(f, dg, x) = f(x) * dg(x)
            pointwise_mul(g, df, x) = g(x) * df(x)
            pointwise_add(pointwise_mul(f, dg), pointwise_mul(g, df), x) =
                pointwise_mul(f, dg, x) + pointwise_mul(g, df, x)
            pointwise_add(pointwise_mul(f, dg), pointwise_mul(g, df), x) =
                f(x) * dg(x) + g(x) * df(x)
            has_derivative_at(pointwise_mul(f, g), x,
                pointwise_add(pointwise_mul(f, dg), pointwise_mul(g, df), x))
        }
    }
}

/// Pointwise squares transport global derivative functions via the product rule.
theorem derivative_fn_square(f: Real -> Real, df: Real -> Real) {
    is_derivative_fn(f, df)
        implies is_derivative_fn(pointwise_mul(f, f), pointwise_add(pointwise_mul(f, df), pointwise_mul(f, df)))
} by {
    if is_derivative_fn(f, df) {
        forall(x: Real) {
            is_derivative_fn_at(f, df, x)
            derivative_pointwise_square(f, x, df(x))
            has_derivative_at(pointwise_mul(f, f), x, f(x) * df(x) + f(x) * df(x))
            pointwise_mul(f, df, x) = f(x) * df(x)
            pointwise_add(pointwise_mul(f, df), pointwise_mul(f, df), x) =
                pointwise_mul(f, df, x) + pointwise_mul(f, df, x)
            pointwise_add(pointwise_mul(f, df), pointwise_mul(f, df), x) =
                f(x) * df(x) + f(x) * df(x)
            has_derivative_at(pointwise_mul(f, f), x,
                pointwise_add(pointwise_mul(f, df), pointwise_mul(f, df), x))
        }
    }
}

/// Composition transports global derivative functions by the chain rule.
theorem derivative_fn_compose(
    f: Real -> Real, g: Real -> Real, df: Real -> Real, dg: Real -> Real
) {
    is_derivative_fn(f, df) and is_derivative_fn(g, dg)
        implies is_derivative_fn(compose(g, f), pointwise_mul(compose(dg, f), df))
} by {
    if is_derivative_fn(f, df) and is_derivative_fn(g, dg) {
        forall(x: Real) {
            is_derivative_fn_at(f, df, x)
            is_derivative_fn_at(g, dg, f(x))
            derivative_compose(f, g, x, df(x), dg(f(x)))
            has_derivative_at(compose(g, f), x, dg(f(x)) * df(x))
            compose(dg, f, x) = dg(f(x))
            pointwise_mul(compose(dg, f), df, x) = compose(dg, f, x) * df(x)
            pointwise_mul(compose(dg, f), df, x) = dg(f(x)) * df(x)
            has_derivative_at(compose(g, f), x, pointwise_mul(compose(dg, f), df, x))
        }
    }
}

/// Differentiability everywhere is preserved by pointwise negation.
theorem differentiable_everywhere_neg(f: Real -> Real) {
    differentiable_everywhere(f) implies differentiable_everywhere(pointwise_neg(f))
} by {
    if differentiable_everywhere(f) {
        forall(x: Real) {
            differentiable_everywhere_at(f, x)
            differentiable_at(f, x)
            differentiable_pointwise_neg(f, x)
            differentiable_at(pointwise_neg(f), x)
        }
    }
}

/// Differentiability everywhere is preserved by pointwise addition.
theorem differentiable_everywhere_add(f: Real -> Real, g: Real -> Real) {
    differentiable_everywhere(f) and differentiable_everywhere(g)
        implies differentiable_everywhere(pointwise_add(f, g))
} by {
    if differentiable_everywhere(f) and differentiable_everywhere(g) {
        forall(x: Real) {
            differentiable_everywhere_at(f, x)
            differentiable_everywhere_at(g, x)
            differentiable_at(f, x)
            differentiable_at(g, x)
            differentiable_pointwise_add(f, g, x)
            differentiable_at(pointwise_add(f, g), x)
        }
    }
}

/// Differentiability everywhere is preserved by pointwise subtraction.
theorem differentiable_everywhere_sub(f: Real -> Real, g: Real -> Real) {
    differentiable_everywhere(f) and differentiable_everywhere(g)
        implies differentiable_everywhere(pointwise_add(f, pointwise_neg(g)))
} by {
    if differentiable_everywhere(f) and differentiable_everywhere(g) {
        forall(x: Real) {
            differentiable_everywhere_at(f, x)
            differentiable_everywhere_at(g, x)
            differentiable_at(f, x)
            differentiable_at(g, x)
            differentiable_pointwise_sub(f, g, x)
            differentiable_at(pointwise_add(f, pointwise_neg(g)), x)
        }
    }
}

/// Differentiability everywhere is preserved by pointwise products.
theorem differentiable_everywhere_mul(f: Real -> Real, g: Real -> Real) {
    differentiable_everywhere(f) and differentiable_everywhere(g)
        implies differentiable_everywhere(pointwise_mul(f, g))
} by {
    if differentiable_everywhere(f) and differentiable_everywhere(g) {
        forall(x: Real) {
            differentiable_everywhere_at(f, x)
            differentiable_everywhere_at(g, x)
            differentiable_at(f, x)
            differentiable_at(g, x)
            differentiable_pointwise_mul(f, g, x)
            differentiable_at(pointwise_mul(f, g), x)
        }
    }
}

/// Differentiability everywhere is preserved by pointwise squaring.
theorem differentiable_everywhere_square(f: Real -> Real) {
    differentiable_everywhere(f) implies differentiable_everywhere(pointwise_mul(f, f))
} by {
    if differentiable_everywhere(f) {
        forall(x: Real) {
            differentiable_everywhere_at(f, x)
            differentiable_at(f, x)
            differentiable_pointwise_square(f, x)
            differentiable_at(pointwise_mul(f, f), x)
        }
    }
}

/// Differentiability everywhere is preserved by composition.
theorem differentiable_everywhere_compose(f: Real -> Real, g: Real -> Real) {
    differentiable_everywhere(f) and differentiable_everywhere(g)
        implies differentiable_everywhere(compose(g, f))
} by {
    if differentiable_everywhere(f) and differentiable_everywhere(g) {
        forall(x: Real) {
            differentiable_everywhere_at(f, x)
            differentiable_everywhere_at(g, f(x))
            differentiable_at(f, x)
            differentiable_at(g, f(x))
            differentiable_compose(f, g, x)
            differentiable_at(compose(g, f), x)
        }
    }
}

/// Differentiability everywhere is preserved by left scalar multiplication.
theorem differentiable_everywhere_const_mul(c: Real, f: Real -> Real) {
    differentiable_everywhere(f)
        implies differentiable_everywhere(pointwise_mul(constant[Real, Real](c), f))
} by {
    if differentiable_everywhere(f) {
        forall(x: Real) {
            differentiable_everywhere_at(f, x)
            differentiable_at(f, x)
            differentiable_pointwise_const_mul(c, f, x)
            differentiable_at(pointwise_mul(constant[Real, Real](c), f), x)
        }
    }
}

/// Differentiability everywhere is preserved by right scalar multiplication.
theorem differentiable_everywhere_mul_const(c: Real, f: Real -> Real) {
    differentiable_everywhere(f)
        implies differentiable_everywhere(pointwise_mul(f, constant[Real, Real](c)))
} by {
    if differentiable_everywhere(f) {
        forall(x: Real) {
            differentiable_everywhere_at(f, x)
            differentiable_at(f, x)
            differentiable_pointwise_mul_const(c, f, x)
            differentiable_at(pointwise_mul(f, constant[Real, Real](c)), x)
        }
    }
}

/// Differentiability everywhere is preserved by linear combinations.
theorem differentiable_everywhere_linear_combination(
    a: Real, b: Real, f: Real -> Real, g: Real -> Real
) {
    differentiable_everywhere(f) and differentiable_everywhere(g) implies differentiable_everywhere(
        pointwise_add(pointwise_mul(constant[Real, Real](a), f), pointwise_mul(constant[Real, Real](b), g))
    )
} by {
    if differentiable_everywhere(f) and differentiable_everywhere(g) {
        forall(x: Real) {
            differentiable_everywhere_at(f, x)
            differentiable_everywhere_at(g, x)
            differentiable_at(f, x)
            differentiable_at(g, x)
            differentiable_pointwise_linear_combination(a, b, f, g, x)
            differentiable_at(
                pointwise_add(pointwise_mul(constant[Real, Real](a), f), pointwise_mul(constant[Real, Real](b), g)),
                x
            )
        }
    }
}

/// Affine real functions have the constant slope as global derivative.
theorem derivative_fn_affine_identity(c: Real, b: Real) {
    is_derivative_fn(
        pointwise_add(pointwise_mul(constant[Real, Real](c), identity_fn[Real]), constant[Real, Real](b)),
        constant[Real, Real](c)
    )
} by {
    forall(x: Real) {
        affine_identity_has_derivative_at(c, b, x)
        constant[Real, Real](c, x) = c
        has_derivative_at(
            pointwise_add(pointwise_mul(constant[Real, Real](c), identity_fn[Real]), constant[Real, Real](b)),
            x,
            constant[Real, Real](c, x)
        )
    }
}

/// Affine real functions are differentiable everywhere.
theorem differentiable_everywhere_affine_identity(c: Real, b: Real) {
    differentiable_everywhere(
        pointwise_add(pointwise_mul(constant[Real, Real](c), identity_fn[Real]), constant[Real, Real](b))
    )
} by {
    forall(x: Real) {
        affine_identity_differentiable_at(c, b, x)
    }
}

/// A function with a global derivative is continuous at every point.
theorem is_derivative_fn_imp_continuous_at(f: Real -> Real, df: Real -> Real, x: Real) {
    is_derivative_fn(f, df) implies continuous_at(f, x)
} by {
    if is_derivative_fn(f, df) {
        is_derivative_fn_at(f, df, x)
        derivative_continuous_at(f, x, df(x))
        continuous_at(f, x)
    }
}

/// A differentiable-everywhere function is continuous at every point.
theorem differentiable_everywhere_imp_continuous_at(f: Real -> Real, x: Real) {
    differentiable_everywhere(f) implies continuous_at(f, x)
} by {
    if differentiable_everywhere(f) {
        differentiable_everywhere(f) = forall(y: Real) { differentiable_at(f, y) }
        differentiable_continuous_at(f, x)
        continuous_at(f, x)
    }
}
