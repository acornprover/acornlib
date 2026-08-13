/// Deep series theory: power-series convergence inside the radius of
/// convergence, documented statements of the root test and Dirichlet's test,
/// and the summation-by-parts identity used by them.
///
/// The fundamental power-series lemma is proved: if Σ a_n x0^n converges at a
/// nonzero x0, then Σ a_n x^n converges absolutely for every x with |x| < |x0|.
/// The proof bounds the terms |a_n x0^n| (a convergent series has bounded
/// terms), then dominates |a_n x^n| by the geometric majorant
/// |x|/|x0|^n times that bound.

from data.basic.functions import function_extensionality
from nat import Nat, pow_zero, pow_add, add_comm, add_sub
from list import partial, sum, map, List
from real.real_base import Real, abs_gte_zero, lte_lt_trans, abs_not_neg, neg_distrib, neg_neg
from real.real_ring import converges, limit, converges_to, mul_abs, mul_nonneg, mul_sub_distrib_right,
    lte_mul_nonneg_right, lt_mul_pos_right, real_mul_comm
from real.real_seq import only_abs_zero_eq_zero, converges_to_unique,
    converges_imp_converges_to, converges_to_imp_converges
from real.real_field import div_le_of_mul_le, mul_div_cancel, zero_is_different_than_one
from real.harmonic import real_one_div_pos
from real.bounded_seq import is_bounded_seq, converges_to_imp_bounded_seq
from real.abs_conv import absolutely_converges, abs_fn
from real.prod_seq import prod_seq
from real.real_series import series_conv_imp_term_vanishes, abs_pow, tail, mul_seq, pow_nonneg, partial_suc,
    partial_mul_seq_comm, partial_add_seq_comm, add_seq, partial_zero, is_lower_bound
from real.series_geometric_majorant import geometric_majorant,
    eventually_abs_le_geometric_absolutely_converges
from order import lt_iff_lte_and_ne, lt_imp_lte

numerals Real

/// The terms of the power series Σ a_n x^n.
define power_series_term(a: Nat -> Real, x: Real, k: Nat) -> Real {
    a(k) * x.pow(k)
}

/// Powers distribute over multiplication: (x · y)^n = x^n · y^n.
theorem pow_mul_distrib(x: Real, y: Real, n: Nat) {
    (x * y).pow(n) = x.pow(n) * y.pow(n)
} by {
    define p(k: Nat) -> Bool {
        (x * y).pow(k) = x.pow(k) * y.pow(k)
    }
    (x * y).pow(Nat.0) = Real.1
    x.pow(Nat.0) = Real.1
    y.pow(Nat.0) = Real.1
    Real.1 * Real.1 = Real.1
    x.pow(Nat.0) * y.pow(Nat.0) = Real.1
    p(Nat.0)
    forall(k: Nat) {
        if p(k) {
            p(k) = ((x * y).pow(k) = x.pow(k) * y.pow(k))
            (x * y).pow(k) = x.pow(k) * y.pow(k)
            (x * y).pow(k.suc) = (x * y) * (x * y).pow(k)
            x.pow(k.suc) = x * x.pow(k)
            y.pow(k.suc) = y * y.pow(k)
            (x * y) * (x.pow(k) * y.pow(k)) = x * (y * (x.pow(k) * y.pow(k)))
            x * (y * (x.pow(k) * y.pow(k))) = x * (y * (y.pow(k) * x.pow(k)))
            x * (y * (y.pow(k) * x.pow(k))) = x * ((y * y.pow(k)) * x.pow(k))
            x * ((y * y.pow(k)) * x.pow(k)) = x * (x.pow(k) * (y * y.pow(k)))
            x * (x.pow(k) * (y * y.pow(k))) = (x * x.pow(k)) * (y * y.pow(k))
            (x * x.pow(k)) * (y * y.pow(k)) = x.pow(k.suc) * y.pow(k.suc)
            (x * y) * (x.pow(k) * y.pow(k)) = (x * x.pow(k)) * (y * y.pow(k))
            (x * y).pow(k.suc) = (x * y) * (x.pow(k) * y.pow(k))
            (x * y).pow(k.suc) = (x * x.pow(k)) * (y * y.pow(k))
            (x * y).pow(k.suc) = x.pow(k.suc) * y.pow(k.suc)
            p(k.suc)
        }
    }
    p(Nat.0) and forall(k: Nat) { p(k) implies p(k.suc) }
    Nat.induction(p)
    p(n)
}

/// If a series converges, its terms form a bounded sequence.
theorem series_conv_imp_terms_bounded(a: Nat -> Real) {
    converges(partial(a)) implies is_bounded_seq(a)
} by {
    if converges(partial(a)) {
        series_conv_imp_term_vanishes(a)
        converges_to(a, Real.0)
        converges_to_imp_bounded_seq(a, Real.0)
        is_bounded_seq(a)
    }
}

/// The absolute value of a nonzero real is positive.
theorem abs_pos_of_ne_zero(x: Real) {
    x != Real.0 implies x.abs > Real.0
} by {
    if x != Real.0 {
        only_abs_zero_eq_zero(x)
        if x.abs = Real.0 {
            x = Real.0
            false
        }
        x.abs != Real.0
        abs_gte_zero(x)
        Real.0 <= x.abs
        lt_iff_lte_and_ne[Real](Real.0, x.abs)
        Real.0 < x.abs = (Real.0 <= x.abs and Real.0 != x.abs)
        Real.0 < x.abs
        x.abs > Real.0
    }
}

/// A positive denominator: a < b and 0 < b imply a / b < 1.
theorem div_lt_one(a: Real, b: Real) {
    a < b and b > Real.0 implies a / b < Real.1
} by {
    if a < b and b > Real.0 {
        b.is_positive
        real_one_div_pos(b)
        (Real.1 / b).is_positive
        lt_mul_pos_right(a, b, Real.1 / b)
        a * (Real.1 / b) < b * (Real.1 / b)
        a * (Real.1 / b) = a / b
        mul_div_cancel(Real.1, b)
        b != Real.0
        b * (Real.1 / b) = Real.1
        a / b < Real.1
    }
}

/// A nonnegative numerator over a positive denominator is nonnegative.
theorem div_nonneg_of_nonneg(a: Real, b: Real) {
    Real.0 <= a and b > Real.0 implies Real.0 <= a / b
} by {
    if Real.0 <= a and b > Real.0 {
        Real.0 * b <= a
        div_le_of_mul_le(Real.0, b, a)
        Real.0 <= a / b
    }
}

/// The absolute value of a power is the power of the absolute value.
theorem abs_pow_abs(x: Real, n: Nat) {
    x.pow(n).abs = x.abs.pow(n)
} by {
    abs_pow(x, n)
    x.abs.pow(n) = x.pow(n).abs
    x.pow(n).abs = x.abs.pow(n)
}

/// The fundamental power-series lemma: if Σ a_n x0^n converges at a nonzero
/// x0, then Σ a_n x^n converges absolutely for every x with |x| < |x0|.
theorem power_series_abs_conv_inside_radius(a: Nat -> Real, x: Real, x0: Real) {
    converges(partial(power_series_term(a, x0))) and x0 != Real.0 and x.abs < x0.abs
    implies absolutely_converges(power_series_term(a, x))
} by {
    if converges(partial(power_series_term(a, x0))) and x0 != Real.0 and x.abs < x0.abs {
        // The terms at x0 are bounded: |a(k) x0^k| < bound for all k.
        series_conv_imp_terms_bounded(power_series_term(a, x0))
        is_bounded_seq(power_series_term(a, x0))
        is_bounded_seq(power_series_term(a, x0)) = exists(b0: Real) {
            forall(n: Nat) {
                power_series_term(a, x0, n).abs < b0
            }
        }
        exists(b0: Real) {
            forall(n: Nat) {
                power_series_term(a, x0, n).abs < b0
            }
        }
        let bound: Real satisfy {
            forall(n: Nat) {
                power_series_term(a, x0, n).abs < bound
            }
        }
        power_series_term(a, x0, Nat.0).abs < bound
        abs_gte_zero(power_series_term(a, x0, Nat.0))
        Real.0 <= power_series_term(a, x0, Nat.0).abs
        lte_lt_trans(Real.0, power_series_term(a, x0, Nat.0).abs, bound)
        Real.0 < bound
        Real.0 <= bound
        // The ratio r = |x| / |x0| satisfies 0 <= r < 1.
        abs_pos_of_ne_zero(x0)
        x0.abs > Real.0
        div_lt_one(x.abs, x0.abs)
        x.abs / x0.abs < Real.1
        div_nonneg_of_nonneg(x.abs, x0.abs)
        Real.0 <= x.abs / x0.abs
        mul_div_cancel(x.abs, x0.abs)
        x0.abs != Real.0
        x0.abs * (x.abs / x0.abs) = x.abs
        real_mul_comm(x.abs / x0.abs, x0.abs)
        (x.abs / x0.abs) * x0.abs = x0.abs * (x.abs / x0.abs)
        (x.abs / x0.abs) * x0.abs = x.abs
        // Domination: |a(k) x^k| <= bound * (|x| / |x0|)^k for every k.
        forall(k: Nat) {
            if Nat.0 <= k {
                mul_abs(a(k), x.pow(k))
                power_series_term(a, x, k) = a(k) * x.pow(k)
                power_series_term(a, x, k).abs = (a(k) * x.pow(k)).abs
                (a(k) * x.pow(k)).abs = a(k).abs * x.pow(k).abs
                abs_pow_abs(x, k)
                x.pow(k).abs = x.abs.pow(k)
                a(k).abs * x.pow(k).abs = a(k).abs * x.abs.pow(k)
                power_series_term(a, x, k).abs = a(k).abs * x.abs.pow(k)
                mul_abs(a(k), x0.pow(k))
                power_series_term(a, x0, k) = a(k) * x0.pow(k)
                power_series_term(a, x0, k).abs = (a(k) * x0.pow(k)).abs
                (a(k) * x0.pow(k)).abs = a(k).abs * x0.pow(k).abs
                abs_pow_abs(x0, k)
                x0.pow(k).abs = x0.abs.pow(k)
                a(k).abs * x0.pow(k).abs = a(k).abs * x0.abs.pow(k)
                power_series_term(a, x0, k).abs = a(k).abs * x0.abs.pow(k)
                pow_mul_distrib(x.abs / x0.abs, x0.abs, k)
                ((x.abs / x0.abs) * x0.abs).pow(k) = (x.abs / x0.abs).pow(k) * x0.abs.pow(k)
                (x.abs / x0.abs) * x0.abs = x.abs
                x.abs.pow(k) = (x.abs / x0.abs).pow(k) * x0.abs.pow(k)
                real_mul_comm(a(k).abs, x.abs.pow(k))
                a(k).abs * x.abs.pow(k) = x.abs.pow(k) * a(k).abs
                x.abs.pow(k) * a(k).abs = ((x.abs / x0.abs).pow(k) * x0.abs.pow(k)) * a(k).abs
                ((x.abs / x0.abs).pow(k) * x0.abs.pow(k)) * a(k).abs = (x.abs / x0.abs).pow(k) * (x0.abs.pow(k) * a(k).abs)
                real_mul_comm(a(k).abs, x0.abs.pow(k))
                a(k).abs * x0.abs.pow(k) = x0.abs.pow(k) * a(k).abs
                (x.abs / x0.abs).pow(k) * (x0.abs.pow(k) * a(k).abs) = (x.abs / x0.abs).pow(k) * (a(k).abs * x0.abs.pow(k))
                power_series_term(a, x0, k).abs * (x.abs / x0.abs).pow(k) = (a(k).abs * x0.abs.pow(k)) * (x.abs / x0.abs).pow(k)
                real_mul_comm(power_series_term(a, x0, k).abs, (x.abs / x0.abs).pow(k))
                (a(k).abs * x0.abs.pow(k)) * (x.abs / x0.abs).pow(k) = power_series_term(a, x0, k).abs * (x.abs / x0.abs).pow(k)
                power_series_term(a, x, k).abs = power_series_term(a, x0, k).abs * (x.abs / x0.abs).pow(k)
                power_series_term(a, x0, k).abs < bound
                lt_imp_lte(power_series_term(a, x0, k).abs, bound)
                power_series_term(a, x0, k).abs <= bound
                Real.0 <= x.abs / x0.abs
                pow_nonneg(x.abs / x0.abs, k)
                Real.0 <= (x.abs / x0.abs).pow(k)
                not (x.abs / x0.abs).pow(k).is_negative
                lte_mul_nonneg_right(power_series_term(a, x0, k).abs, bound, (x.abs / x0.abs).pow(k))
                power_series_term(a, x0, k).abs * (x.abs / x0.abs).pow(k) <= bound * (x.abs / x0.abs).pow(k)
                power_series_term(a, x, k).abs <= bound * (x.abs / x0.abs).pow(k)
                geometric_majorant(bound, x.abs / x0.abs, k) = bound * (x.abs / x0.abs).pow(k)
                bound * (x.abs / x0.abs).pow(k) = geometric_majorant(bound, x.abs / x0.abs, k)
                power_series_term(a, x, k).abs <= geometric_majorant(bound, x.abs / x0.abs, k)
                abs_fn(power_series_term(a, x), k) = power_series_term(a, x, k).abs
                abs_fn(power_series_term(a, x))(k) <= geometric_majorant(bound, x.abs / x0.abs, k)
            }
        }
        forall(k: Nat) {
            Nat.0 <= k implies abs_fn(power_series_term(a, x))(k) <= geometric_majorant(bound, x.abs / x0.abs, k)
        }
        eventually_abs_le_geometric_absolutely_converges(
            power_series_term(a, x), bound, x.abs / x0.abs, Nat.0)
        absolutely_converges(power_series_term(a, x))
    }
}

// ---------------------------------------------------------------------------
// The root test and Dirichlet's (Abel's) test.
//
// Both are classical convergence tests that the library's machinery supports
// only partially; their full statements are recorded here with the intended
// proof routes, and the parts that are already provable are proved below.
//
// Root test: if limsup |a_n|^(1/n) < 1 then Σ a_n converges absolutely.
// The library has the limit superior (analysis.real.real_limsup.limsup) and a
// geometric-majorant consumer (eventually_abs_le_geometric_absolutely_converges),
// but the n-th root |a_n|^(1/n) is Option-valued (real.rpow), so the root
// sequence is a choice over that option and the root-power identity
// (x^(1/n))^n = x is not yet exposed.  The provable geometric form is:
// if |a_n| <= c · r^n eventually with 0 <= r < 1, then Σ a_n converges
// absolutely — exactly the statement of
// real.series_geometric_majorant.eventually_abs_le_geometric_absolutely_converges.
//
// Dirichlet's test: if (b_n) is decreasing to zero and the partial sums of
// (a_n) are bounded, then Σ a_n b_n converges.  The proof route is the
// summation-by-parts identity (windowed form), for m <= n:
//
//     sum_{k=m}^{n-1} a_k b_k = S_n b_{n-1} - S_m b_m + sum_{k=m+1}^{n-1} S_k (b_{k-1} - b_k)
//
// where S_k = partial(a, k).  With |S_k| <= M and 0 <= b decreasing to zero,
// the window is bounded by 3M · b_m, which vanishes as m grows, so the
// windowed Cauchy criterion (real.series_cauchy.is_cauchy_series) applies.
// The identity requires reindexing finite sums over `until` ranges, which the
// library does not yet automate; it is recorded here as future work.

/// The summation-by-parts step term S_{k+1} (g(k+1) - g(k)), where S is the
/// partial-sum sequence of f.
define sbp_step(f: Nat -> Real, g: Nat -> Real, k: Nat) -> Real {
    partial(f, k.suc) * (g(k.suc) - g(k))
}

/// x − (x − y) = y.
theorem sub_sub_cancel(x: Real, y: Real) {
    x - (x - y) = y
} by {
    x - (x - y) = x + -(x - y)
    x - y = x + -y
    -(x - y) = -(x + -y)
    neg_distrib(x, -y)
    -(x + -y) = -x + -(-y)
    neg_neg(y)
    -(-y) = y
    -x + -(-y) = -x + y
    x + (-x + y) = x + -x + y
    x + -x = Real.0
    Real.0 + y = y
    x - (x - y) = y
}

/// a·b − a·(b − c) = a·c.
theorem mul_sub_cancel(a: Real, b: Real, c: Real) {
    a * b - a * (b - c) = a * c
} by {
    mul_sub_distrib_right(a, b, c)
    a * (b - c) = a * b - a * c
    a * b - (a * b - a * c) = a * c
    sub_sub_cancel(a * b, a * c)
}

/// (a + b) − (c + d) = (a − c) + (b − d).
theorem sub_add_rearrange(a: Real, b: Real, c: Real, d: Real) {
    a + b - (c + d) = (a - c) + (b - d)
} by {
    a + b - (c + d) = a + b + -(c + d)
    neg_distrib(c, d)
    -(c + d) = -c + -d
    a + b + (-c + -d) = a + b + -c + -d
    a + b + -c = a + (b + -c)
    a + (b + -c) = a + (-c + b)
    a + (-c + b) = a + -c + b
    a + b + -c = a + -c + b
    (a + b + -c) + -d = (a + -c + b) + -d
    a + b + -c + -d = a + -c + b + -d
    a + -c = a - c
    b + -d = b - d
    a + -c + b + -d = (a - c) + (b - d)
    a + b - (c + d) = (a - c) + (b - d)
}

/// The summation-by-parts (Abel) identity for finite sums:
/// Σ_{k<n} f(k) g(k) = S_n g(n) - Σ_{k<n} S_{k+1} (g(k+1) - g(k)),
/// where S_n = partial(f, n).
theorem summation_by_parts(f: Nat -> Real, g: Nat -> Real, n: Nat) {
    partial(prod_seq(f, g), n) =
        partial(f, n) * g(n) - sum(map(n.range, sbp_step(f, g)))
} by {
    define p(k: Nat) -> Bool {
        partial(prod_seq(f, g), k) =
            partial(f, k) * g(k) - sum(map(k.range, sbp_step(f, g)))
    }
    // Base case: k = 0.
    Nat.0.range = List.nil[Nat]
    map(Nat.0.range, sbp_step(f, g)) = List.nil[Real]
    sum(map(Nat.0.range, sbp_step(f, g))) = Real.0
    partial(prod_seq(f, g), Nat.0) = Real.0
    partial(f, Nat.0) = Real.0
    partial(f, Nat.0) * g(Nat.0) = Real.0
    partial(f, Nat.0) * g(Nat.0) - sum(map(Nat.0.range, sbp_step(f, g))) = Real.0
    p(Nat.0)
    // Inductive step: from k to k.suc.
    forall(m: Nat) {
        if p(m) {
            p(m) = (partial(prod_seq(f, g), m) =
                partial(f, m) * g(m) - sum(map(m.range, sbp_step(f, g))))
            partial(prod_seq(f, g), m) =
                partial(f, m) * g(m) - sum(map(m.range, sbp_step(f, g)))
            partial_suc(prod_seq(f, g), m)
            partial(prod_seq(f, g), m.suc) = partial(prod_seq(f, g), m) + prod_seq(f, g)(m)
            prod_seq(f, g, m) = f(m) * g(m)
            partial(prod_seq(f, g), m.suc) =
                (partial(f, m) * g(m) - sum(map(m.range, sbp_step(f, g)))) + f(m) * g(m)
            // S_{m+1} = S_m + f(m).
            partial_suc(f, m)
            partial(f, m.suc) = partial(f, m) + f(m)
            sbp_step(f, g, m) = partial(f, m.suc) * (g(m.suc) - g(m))
            // The sum over m.suc.range is the sum over m.range plus the new step,
            // since partial sums step by one term.
            partial_suc(sbp_step(f, g), m)
            partial(sbp_step(f, g), m.suc) = partial(sbp_step(f, g), m) + sbp_step(f, g, m)
            sum(map(m.suc.range, sbp_step(f, g))) = partial(sbp_step(f, g), m.suc)
            sum(map(m.range, sbp_step(f, g))) = partial(sbp_step(f, g), m)
            sum(map(m.suc.range, sbp_step(f, g))) =
                sum(map(m.range, sbp_step(f, g))) + sbp_step(f, g, m)
            // Expand the target of the step:
            // S_{m+1} g(m+1) - (Σ + S_{m+1}(g(m+1) - g(m))) = S_m g(m) + f(m) g(m) - Σ.
            partial(f, m.suc) * g(m.suc) = (partial(f, m) + f(m)) * g(m.suc)
            (partial(f, m) + f(m)) * g(m.suc) = partial(f, m) * g(m.suc) + f(m) * g(m.suc)
            (partial(f, m) + f(m)) * (g(m.suc) - g(m)) =
                partial(f, m) * (g(m.suc) - g(m)) + f(m) * (g(m.suc) - g(m))
            partial(f, m.suc) * g(m.suc) - (sum(map(m.range, sbp_step(f, g))) + sbp_step(f, g, m)) =
                partial(f, m) * g(m.suc) + f(m) * g(m.suc) - sum(map(m.range, sbp_step(f, g))) - sbp_step(f, g, m)
            partial(f, m.suc) * g(m.suc) - sum(map(m.suc.range, sbp_step(f, g))) =
                partial(f, m.suc) * g(m.suc) - (sum(map(m.range, sbp_step(f, g))) + sbp_step(f, g, m))
            partial(f, m.suc) * g(m.suc) - sum(map(m.suc.range, sbp_step(f, g))) =
                partial(f, m) * g(m.suc) + f(m) * g(m.suc) - sum(map(m.range, sbp_step(f, g))) - sbp_step(f, g, m)
            partial(f, m) * g(m.suc) + f(m) * g(m.suc) - sum(map(m.range, sbp_step(f, g))) - sbp_step(f, g, m) =
                partial(f, m) * g(m.suc) + f(m) * g(m.suc) - sum(map(m.range, sbp_step(f, g))) - (partial(f, m) + f(m)) * (g(m.suc) - g(m))
            mul_sub_cancel(partial(f, m), g(m.suc), g(m))
            partial(f, m) * g(m.suc) - partial(f, m) * (g(m.suc) - g(m)) =
                partial(f, m) * g(m)
            mul_sub_cancel(f(m), g(m.suc), g(m))
            f(m) * g(m.suc) - f(m) * (g(m.suc) - g(m)) =
                f(m) * g(m)
            sub_add_rearrange(
                partial(f, m) * g(m.suc),
                f(m) * g(m.suc),
                partial(f, m) * (g(m.suc) - g(m)),
                f(m) * (g(m.suc) - g(m)))
            partial(f, m) * g(m.suc) + f(m) * g(m.suc) - (partial(f, m) * (g(m.suc) - g(m)) + f(m) * (g(m.suc) - g(m))) =
                (partial(f, m) * g(m.suc) - partial(f, m) * (g(m.suc) - g(m))) + (f(m) * g(m.suc) - f(m) * (g(m.suc) - g(m)))
            (partial(f, m) * g(m.suc) - partial(f, m) * (g(m.suc) - g(m))) + (f(m) * g(m.suc) - f(m) * (g(m.suc) - g(m))) =
                partial(f, m) * g(m) + f(m) * g(m)
            partial(f, m) * g(m.suc) + f(m) * g(m.suc) - sum(map(m.range, sbp_step(f, g))) - sbp_step(f, g, m) =
                partial(f, m) * g(m) + f(m) * g(m) - sum(map(m.range, sbp_step(f, g)))
            partial(f, m.suc) * g(m.suc) - sum(map(m.suc.range, sbp_step(f, g))) =
                partial(f, m) * g(m) + f(m) * g(m) - sum(map(m.range, sbp_step(f, g)))
            (partial(f, m) * g(m) - sum(map(m.range, sbp_step(f, g)))) + f(m) * g(m) =
                partial(f, m) * g(m) + f(m) * g(m) - sum(map(m.range, sbp_step(f, g)))
            partial(prod_seq(f, g), m.suc) =
                partial(f, m.suc) * g(m.suc) - sum(map(m.suc.range, sbp_step(f, g)))
            p(m.suc)
        }
    }
    p(Nat.0) and forall(m: Nat) { p(m) implies p(m.suc) }
    Nat.induction(p)
    p(n)
}
