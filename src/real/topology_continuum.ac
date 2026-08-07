from order_set import closed_interval_set, closed_interval_set_contains_lower
from data.basic.set import Set, singleton_set_is_not_empty
from real.real_field import Real
from real.topology import closure, is_bounded_real_set, is_closed_set, is_connected_real_set
from real.topology_bounded_connected import compact_connected_real_set_is_bounded_connected,
    is_bounded_connected_real_set
from real.topology_compact import compact_real_set_eq_closure, compact_real_set_is_bounded,
    compact_real_set_is_closed, closure_of_compact_real_set_is_compact,
    intersection_of_compact_real_sets_is_compact, is_compact_real_set,
    singleton_real_set_is_compact
from real.topology_connected_algebra import connected_real_set_of_eq,
    intersection_of_connected_real_sets_is_connected, singleton_real_set_is_connected
from real.topology_connected_intervals import closed_interval_set_is_connected
from real.topology_intervals import closed_interval_set_is_compact

/// True if a real set is nonempty, compact, and connected.
define is_continuum_real_set(s: Set[Real]) -> Bool {
    not s.is_empty and is_compact_real_set(s) and is_connected_real_set(s)
}

/// A continuum real set is nonempty.
theorem continuum_real_set_is_nonempty(s: Set[Real]) {
    is_continuum_real_set(s) implies not s.is_empty
} by {
    if is_continuum_real_set(s) {
        is_continuum_real_set(s) = (not s.is_empty and is_compact_real_set(s) and is_connected_real_set(s))
        not s.is_empty
    }
}

/// A continuum real set is compact.
theorem continuum_real_set_is_compact(s: Set[Real]) {
    is_continuum_real_set(s) implies is_compact_real_set(s)
} by {
    if is_continuum_real_set(s) {
        is_continuum_real_set(s) = (not s.is_empty and is_compact_real_set(s) and is_connected_real_set(s))
        is_compact_real_set(s)
    }
}

/// A continuum real set is connected.
theorem continuum_real_set_is_connected(s: Set[Real]) {
    is_continuum_real_set(s) implies is_connected_real_set(s)
} by {
    if is_continuum_real_set(s) {
        is_continuum_real_set(s) = (not s.is_empty and is_compact_real_set(s) and is_connected_real_set(s))
        is_connected_real_set(s)
    }
}

/// A nonempty compact connected real set is a continuum real set.
theorem continuum_real_set_intro(s: Set[Real]) {
    not s.is_empty and is_compact_real_set(s) and is_connected_real_set(s) implies is_continuum_real_set(s)
}

/// A continuum real set is closed.
theorem continuum_real_set_is_closed(s: Set[Real]) {
    is_continuum_real_set(s) implies is_closed_set(s)
} by {
    if is_continuum_real_set(s) {
        continuum_real_set_is_compact(s)
        is_compact_real_set(s)
        compact_real_set_is_closed(s)
        is_closed_set(s)
    }
}

/// A continuum real set is bounded.
theorem continuum_real_set_is_bounded(s: Set[Real]) {
    is_continuum_real_set(s) implies is_bounded_real_set(s)
} by {
    if is_continuum_real_set(s) {
        continuum_real_set_is_compact(s)
        is_compact_real_set(s)
        compact_real_set_is_bounded(s)
        is_bounded_real_set(s)
    }
}

/// A continuum real set is bounded connected.
theorem continuum_real_set_is_bounded_connected(s: Set[Real]) {
    is_continuum_real_set(s) implies is_bounded_connected_real_set(s)
} by {
    if is_continuum_real_set(s) {
        continuum_real_set_is_compact(s)
        continuum_real_set_is_connected(s)
        is_compact_real_set(s)
        is_connected_real_set(s)
        compact_connected_real_set_is_bounded_connected(s)
        is_bounded_connected_real_set(s)
    }
}

/// A singleton real set is a continuum real set.
theorem singleton_real_set_is_continuum(a: Real) {
    is_continuum_real_set(Set[Real].singleton(a))
} by {
    singleton_set_is_not_empty[Real](a)
    singleton_real_set_is_compact(a)
    singleton_real_set_is_connected(a)
    not (Set[Real].singleton(a)).is_empty
    is_compact_real_set(Set[Real].singleton(a))
    is_connected_real_set(Set[Real].singleton(a))
    is_continuum_real_set(Set[Real].singleton(a))
}

/// A nonempty closed interval is a continuum real set.
theorem closed_interval_set_is_continuum(lower: Real, upper: Real) {
    lower <= upper implies is_continuum_real_set(closed_interval_set(lower, upper))
} by {
    if lower <= upper {
        closed_interval_set_contains_lower[Real](lower, upper)
        closed_interval_set(lower, upper).contains(lower)
        not closed_interval_set(lower, upper).is_empty
        closed_interval_set_is_compact(lower, upper)
        closed_interval_set_is_connected(lower, upper)
        is_compact_real_set(closed_interval_set(lower, upper))
        is_connected_real_set(closed_interval_set(lower, upper))
        is_continuum_real_set(closed_interval_set(lower, upper))
    }
}

/// A nonempty intersection of two continuum real sets is a continuum real set.
theorem intersection_of_continuum_real_sets_is_continuum(s: Set[Real], t: Set[Real]) {
    is_continuum_real_set(s) and is_continuum_real_set(t) and not s.intersection(t).is_empty
    implies is_continuum_real_set(s.intersection(t))
} by {
    if is_continuum_real_set(s) and is_continuum_real_set(t) and not s.intersection(t).is_empty {
        continuum_real_set_is_compact(s)
        continuum_real_set_is_compact(t)
        continuum_real_set_is_connected(s)
        continuum_real_set_is_connected(t)
        is_compact_real_set(s)
        is_compact_real_set(t)
        is_connected_real_set(s)
        is_connected_real_set(t)
        intersection_of_compact_real_sets_is_compact(s, t)
        intersection_of_connected_real_sets_is_connected(s, t)
        is_compact_real_set(s.intersection(t))
        is_connected_real_set(s.intersection(t))
        not s.intersection(t).is_empty
        is_continuum_real_set(s.intersection(t))
    }
}

/// A continuum real set is equal to its closure.
theorem continuum_real_set_eq_closure(s: Set[Real]) {
    is_continuum_real_set(s) implies s = closure(s)
} by {
    if is_continuum_real_set(s) {
        continuum_real_set_is_compact(s)
        is_compact_real_set(s)
        compact_real_set_eq_closure(s)
        s = closure(s)
    }
}

/// The closure of a continuum real set is a continuum real set.
theorem closure_of_continuum_real_set_is_continuum(s: Set[Real]) {
    is_continuum_real_set(s) implies is_continuum_real_set(closure(s))
} by {
    if is_continuum_real_set(s) {
        continuum_real_set_is_compact(s)
        continuum_real_set_is_connected(s)
        is_compact_real_set(s)
        is_connected_real_set(s)
        closure_of_compact_real_set_is_compact(s)
        is_compact_real_set(closure(s))
        continuum_real_set_eq_closure(s)
        s = closure(s)
        continuum_real_set_is_nonempty(s)
        not s.is_empty
        not closure(s).is_empty
        connected_real_set_of_eq(s, closure(s))
        is_connected_real_set(closure(s))
        is_continuum_real_set(closure(s))
    }
}

/// The closure of a continuum real set is connected.
theorem closure_of_continuum_real_set_is_connected(s: Set[Real]) {
    is_continuum_real_set(s) implies is_connected_real_set(closure(s))
} by {
    if is_continuum_real_set(s) {
        closure_of_continuum_real_set_is_continuum(s)
        is_continuum_real_set(closure(s))
        continuum_real_set_is_connected(closure(s))
        is_connected_real_set(closure(s))
    }
}

/// The closure of a continuum real set is compact.
theorem closure_of_continuum_real_set_is_compact(s: Set[Real]) {
    is_continuum_real_set(s) implies is_compact_real_set(closure(s))
} by {
    if is_continuum_real_set(s) {
        closure_of_continuum_real_set_is_continuum(s)
        is_continuum_real_set(closure(s))
        continuum_real_set_is_compact(closure(s))
        is_compact_real_set(closure(s))
    }
}

/// The intersection of two continuum real sets is bounded connected.
theorem intersection_of_continuum_real_sets_is_bounded_connected(s: Set[Real], t: Set[Real]) {
    is_continuum_real_set(s) and is_continuum_real_set(t)
    implies is_bounded_connected_real_set(s.intersection(t))
} by {
    if is_continuum_real_set(s) and is_continuum_real_set(t) {
        continuum_real_set_is_compact(s)
        continuum_real_set_is_compact(t)
        continuum_real_set_is_connected(s)
        continuum_real_set_is_connected(t)
        is_compact_real_set(s)
        is_compact_real_set(t)
        is_connected_real_set(s)
        is_connected_real_set(t)
        intersection_of_compact_real_sets_is_compact(s, t)
        intersection_of_connected_real_sets_is_connected(s, t)
        is_compact_real_set(s.intersection(t))
        is_connected_real_set(s.intersection(t))
        compact_connected_real_set_is_bounded_connected(s.intersection(t))
        is_bounded_connected_real_set(s.intersection(t))
    }
}

/// The closure of a continuum real set is bounded connected.
theorem closure_of_continuum_real_set_is_bounded_connected(s: Set[Real]) {
    is_continuum_real_set(s) implies is_bounded_connected_real_set(closure(s))
} by {
    if is_continuum_real_set(s) {
        closure_of_continuum_real_set_is_continuum(s)
        is_continuum_real_set(closure(s))
        continuum_real_set_is_bounded_connected(closure(s))
        is_bounded_connected_real_set(closure(s))
    }
}
