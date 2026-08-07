from data.basic.functions import compose, function_extensionality
from real.continuity_affine import affine_real, continuous_affine_real,
    continuous_at_affine_real
from real.continuity_algebra import compose_continuous, compose_continuous_at
from real.continuity_composition import continuous_imp_continuous_at
from real.continuity_base import Real, continuous, continuous_at

/// The shift of a real function by a real constant h, mapping x to f(x + h).
define shift_fn(f: Real -> Real, h: Real, x: Real) -> Real {
    f(x + h)
}

/// The shift function agrees with the composition of f and the affine function x -> x + h.
theorem shift_fn_eq_compose_affine(f: Real -> Real, h: Real) {
    shift_fn(f, h) = compose(f, affine_real(Real.1, h))
} by {
    forall(x: Real) {
        affine_real(Real.1, h, x) = Real.1 * x + h
        Real.1 * x = x
        affine_real(Real.1, h, x) = x + h
        compose(f, affine_real(Real.1, h), x) = f(affine_real(Real.1, h, x))
        compose(f, affine_real(Real.1, h), x) = f(x + h)
        shift_fn(f, h, x) = f(x + h)
        shift_fn(f, h, x) = compose(f, affine_real(Real.1, h), x)
    }
    function_extensionality(shift_fn(f, h), compose(f, affine_real(Real.1, h)))
}

/// The shift of a function continuous at x + h by h is continuous at x.
theorem continuous_at_shift_fn(f: Real -> Real, h: Real, x: Real) {
    continuous_at(f, x + h) implies continuous_at(shift_fn(f, h), x)
} by {
    if continuous_at(f, x + h) {
        continuous_at_affine_real(Real.1, h, x)
        continuous_at(affine_real(Real.1, h), x)
        affine_real(Real.1, h, x) = Real.1 * x + h
        Real.1 * x = x
        affine_real(Real.1, h, x) = x + h
        continuous_at(f, affine_real(Real.1, h, x))
        compose_continuous_at(f, affine_real(Real.1, h), x)
        continuous_at(compose(f, affine_real(Real.1, h)), x)
        shift_fn_eq_compose_affine(f, h)
        continuous_at(shift_fn(f, h), x)
    }
}

/// The shift of a continuous real function by a real constant is continuous.
theorem continuous_shift_fn(f: Real -> Real, h: Real) {
    continuous(f) implies continuous(shift_fn(f, h))
} by {
    if continuous(f) {
        continuous_affine_real(Real.1, h)
        continuous(affine_real(Real.1, h))
        compose_continuous(f, affine_real(Real.1, h))
        continuous(compose(f, affine_real(Real.1, h)))
        shift_fn_eq_compose_affine(f, h)
        continuous(shift_fn(f, h))
    }
}
