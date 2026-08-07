from data.basic.set import Set, compl_contains_eq, intersection_contains_eq, intersection_contains_intro,
    union_contains_eq, union_contains_left, union_contains_right
from real.real_field import Real
from real.topology import adherent_point_eps, adherent_point_intro,
    closed_set_contains_adherent, eps_adherent_of_contains_close, interior_point_intro,
    is_adherent_point_of_set, is_closed_set, is_eps_adherent_to_set,
    is_interior_point, is_open_set, open_set_interior_point

numerals Real

/// Epsilon-adherence to an intersection gives epsilon-adherence to the left set.
theorem eps_adherent_intersection_left(s: Set[Real], t: Set[Real], x: Real, eps: Real) {
    is_eps_adherent_to_set(s.intersection(t), x, eps) implies is_eps_adherent_to_set(s, x, eps)
} by {
    if is_eps_adherent_to_set(s.intersection(t), x, eps) {
        let y: Real satisfy {
            s.intersection(t).contains(y) and y.is_close(x, eps)
        }
        intersection_contains_eq(s, t, y)
        s.contains(y)
        eps_adherent_of_contains_close(s, x, y, eps)
        is_eps_adherent_to_set(s, x, eps)
    }
}

/// Epsilon-adherence to an intersection gives epsilon-adherence to the right set.
theorem eps_adherent_intersection_right(s: Set[Real], t: Set[Real], x: Real, eps: Real) {
    is_eps_adherent_to_set(s.intersection(t), x, eps) implies is_eps_adherent_to_set(t, x, eps)
} by {
    if is_eps_adherent_to_set(s.intersection(t), x, eps) {
        let y: Real satisfy {
            s.intersection(t).contains(y) and y.is_close(x, eps)
        }
        intersection_contains_eq(s, t, y)
        t.contains(y)
        eps_adherent_of_contains_close(t, x, y, eps)
        is_eps_adherent_to_set(t, x, eps)
    }
}

/// Adherence to an intersection gives adherence to the left set.
theorem adherent_intersection_left(s: Set[Real], t: Set[Real], x: Real) {
    is_adherent_point_of_set(s.intersection(t), x) implies is_adherent_point_of_set(s, x)
} by {
    if is_adherent_point_of_set(s.intersection(t), x) {
        forall(eps: Real) {
            if eps.is_positive {
                adherent_point_eps(s.intersection(t), x, eps)
                is_eps_adherent_to_set(s.intersection(t), x, eps)
                eps_adherent_intersection_left(s, t, x, eps)
                is_eps_adherent_to_set(s, x, eps)
            }
        }
        adherent_point_intro(s, x)
        is_adherent_point_of_set(s, x)
    }
}

/// Adherence to an intersection gives adherence to the right set.
theorem adherent_intersection_right(s: Set[Real], t: Set[Real], x: Real) {
    is_adherent_point_of_set(s.intersection(t), x) implies is_adherent_point_of_set(t, x)
} by {
    if is_adherent_point_of_set(s.intersection(t), x) {
        forall(eps: Real) {
            if eps.is_positive {
                adherent_point_eps(s.intersection(t), x, eps)
                is_eps_adherent_to_set(s.intersection(t), x, eps)
                eps_adherent_intersection_right(s, t, x, eps)
                is_eps_adherent_to_set(t, x, eps)
            }
        }
        adherent_point_intro(t, x)
        is_adherent_point_of_set(t, x)
    }
}

/// An adherent point of two closed sets lies in their intersection.
theorem closed_intersection_contains_adherent(s: Set[Real], t: Set[Real], x: Real) {
    is_closed_set(s) and is_closed_set(t) and is_adherent_point_of_set(s.intersection(t), x)
    implies s.intersection(t).contains(x)
} by {
    if is_closed_set(s) and is_closed_set(t) and is_adherent_point_of_set(s.intersection(t), x) {
        adherent_intersection_left(s, t, x)
        is_adherent_point_of_set(s, x)
        closed_set_contains_adherent(s, x)
        s.contains(x)
        adherent_intersection_right(s, t, x)
        is_adherent_point_of_set(t, x)
        closed_set_contains_adherent(t, x)
        t.contains(x)
        intersection_contains_intro(s, t, x)
        s.intersection(t).contains(x)
    }
}

/// The intersection of two closed sets is closed.
theorem intersection_of_closed_is_closed(s: Set[Real], t: Set[Real]) {
    is_closed_set(s) and is_closed_set(t) implies is_closed_set(s.intersection(t))
} by {
    if is_closed_set(s) and is_closed_set(t) {
        forall(x: Real) {
            if is_adherent_point_of_set(s.intersection(t), x) {
                closed_intersection_contains_adherent(s, t, x)
            }
        }
        is_closed_set(s.intersection(t))
    }
}

/// The complement of an open set is closed.
theorem complement_of_open_is_closed(s: Set[Real]) {
    is_open_set(s) implies is_closed_set(s.c)
} by {
    if is_open_set(s) {
        forall(x: Real) {
            if is_adherent_point_of_set(s.c, x) {
                if s.contains(x) {
                    open_set_interior_point(s, x)
                    is_interior_point(s, x)
                    let eps: Real satisfy {
                        eps.is_positive and forall(y: Real) {
                            y.is_close(x, eps) implies s.contains(y)
                        }
                    }
                    adherent_point_eps(s.c, x, eps)
                    is_eps_adherent_to_set(s.c, x, eps)
                    let y: Real satisfy {
                        s.c.contains(y) and y.is_close(x, eps)
                    }
                    compl_contains_eq(s, y)
                    not s.contains(y)
                    s.contains(y)
                    false
                }
                not s.contains(x)
                compl_contains_eq(s, x)
                s.c.contains(x)
            }
        }
    }
}

/// The union of two open sets is open.
theorem union_of_open_is_open(s: Set[Real], t: Set[Real]) {
    is_open_set(s) and is_open_set(t) implies is_open_set(s.union(t))
} by {
    if is_open_set(s) and is_open_set(t) {
        forall(x: Real) {
            if s.union(t).contains(x) {
                union_contains_eq(s, t, x)
                if s.contains(x) {
                    open_set_interior_point(s, x)
                    is_interior_point(s, x)
                    let eps: Real satisfy {
                        eps.is_positive and forall(y: Real) {
                            y.is_close(x, eps) implies s.contains(y)
                        }
                    }
                    forall(y: Real) {
                        if y.is_close(x, eps) {
                            s.contains(y)
                            union_contains_left(s, t, y)
                        }
                    }
                    interior_point_intro(s.union(t), x, eps)
                    is_interior_point(s.union(t), x)
                } else {
                    t.contains(x)
                    open_set_interior_point(t, x)
                    is_interior_point(t, x)
                    let eps: Real satisfy {
                        eps.is_positive and forall(y: Real) {
                            y.is_close(x, eps) implies t.contains(y)
                        }
                    }
                    forall(y: Real) {
                        if y.is_close(x, eps) {
                            t.contains(y)
                            union_contains_right(s, t, y)
                        }
                    }
                    interior_point_intro(s.union(t), x, eps)
                    is_interior_point(s.union(t), x)
                }
            }
        }
        is_open_set(s.union(t))
    }
}
