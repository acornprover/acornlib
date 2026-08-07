from nat import Nat, lte_trans
from data.basic.set import Set, intersection_contains_intro, subset_contains, subset_refl, subset_trans
from real.real_field import Real
from real.sequence_set_membership import seq_eventually_in_real_set,
    seq_frequently_in_real_set, seq_in_real_set

/// True if a real number occurs in a sequence.
define sequence_range_contains(a: Nat -> Real, x: Real) -> Bool {
    exists(n: Nat) {
        x = a(n)
    }
}

/// True if a real number occurs in the tail of a sequence starting at `m`.
define sequence_tail_contains(a: Nat -> Real, m: Nat, x: Real) -> Bool {
    exists(n: Nat) {
        m <= n and x = a(n)
    }
}

/// The range of a real sequence as a set.
define sequence_range_set(a: Nat -> Real) -> Set[Real] {
    Set[Real].new(sequence_range_contains(a))
}

/// The tail range of a real sequence as a set.
define sequence_tail_set(a: Nat -> Real, m: Nat) -> Set[Real] {
    Set[Real].new(sequence_tail_contains(a, m))
}

/// True if the tail of a sequence starting at `m` meets a set.
define sequence_tail_meets_set(a: Nat -> Real, m: Nat, s: Set[Real]) -> Bool {
    exists(x: Real) {
        sequence_tail_set(a, m).contains(x) and s.contains(x)
    }
}

/// Membership in a sequence range is occurrence as a sequence term.
theorem sequence_range_set_contains_eq(a: Nat -> Real, x: Real) {
    sequence_range_set(a).contains(x) = exists(n: Nat) { x = a(n) }
}

/// Membership in a sequence tail is occurrence as a sufficiently late term.
theorem sequence_tail_set_contains_eq(a: Nat -> Real, m: Nat, x: Real) {
    sequence_tail_set(a, m).contains(x) = exists(n: Nat) { m <= n and x = a(n) }
}

/// Every term belongs to the range of the sequence.
theorem sequence_range_set_contains_term(a: Nat -> Real, n: Nat) {
    sequence_range_set(a).contains(a(n))
} by {
    exists(k: Nat) {
        a(n) = a(k)
    }
}

/// Every sufficiently late term belongs to the corresponding tail.
theorem sequence_tail_set_contains_term(a: Nat -> Real, m: Nat, n: Nat) {
    m <= n implies sequence_tail_set(a, m).contains(a(n))
} by {
    if m <= n {
        exists(k: Nat) {
            m <= k and a(n) = a(k)
        }
    }
}

/// Every term of a tail belongs to the range.
theorem sequence_tail_set_subset_range_set(a: Nat -> Real, m: Nat) {
    sequence_tail_set(a, m).subset(sequence_range_set(a))
} by {
    forall(x: Real) {
        if sequence_tail_set(a, m).contains(x) {
            let n: Nat satisfy {
                m <= n and x = a(n)
            }
            exists(k: Nat) {
                x = a(k)
            }
            sequence_range_set(a).contains(x)
        }
    }
}

/// Later tails are contained in earlier tails.
theorem sequence_tail_set_subset_of_le(a: Nat -> Real, m: Nat, n: Nat) {
    m <= n implies sequence_tail_set(a, n).subset(sequence_tail_set(a, m))
} by {
    if m <= n {
        forall(x: Real) {
            if sequence_tail_set(a, n).contains(x) {
                let k: Nat satisfy {
                    n <= k and x = a(k)
                }
                m <= k
                exists(j: Nat) {
                    m <= j and x = a(j)
                }
                sequence_tail_set(a, m).contains(x)
            }
        }
    }
}

/// Each tail is contained in itself.
theorem sequence_tail_set_subset_self(a: Nat -> Real, m: Nat) {
    sequence_tail_set(a, m).subset(sequence_tail_set(a, m))
} by {
    subset_refl[Real](sequence_tail_set(a, m))
}

/// Containment in a tail is monotone toward earlier starts.
theorem sequence_tail_set_contains_of_le(a: Nat -> Real, m: Nat, n: Nat, x: Real) {
    m <= n and sequence_tail_set(a, n).contains(x) implies sequence_tail_set(a, m).contains(x)
} by {
    if m <= n and sequence_tail_set(a, n).contains(x) {
        sequence_tail_set_subset_of_le(a, m, n)
        sequence_tail_set(a, n).subset(sequence_tail_set(a, m))
        subset_contains(sequence_tail_set(a, n), sequence_tail_set(a, m), x)
        sequence_tail_set(a, m).contains(x)
    }
}

/// The sequence is contained in its range.
theorem sequence_range_set_contains_sequence(a: Nat -> Real) {
    seq_in_real_set(sequence_range_set(a), a)
} by {
    forall(n: Nat) {
        sequence_range_set_contains_term(a, n)
    }
}

/// A sequence is frequently in its range.
theorem sequence_range_set_frequently_contains_sequence(a: Nat -> Real) {
    seq_frequently_in_real_set(sequence_range_set(a), a)
} by {
    sequence_range_set_contains_sequence(a)
    seq_in_real_set(sequence_range_set(a), a)
    from real.sequence_set_membership import seq_in_real_set_frequently
    seq_in_real_set_frequently(sequence_range_set(a), a)
    seq_frequently_in_real_set(sequence_range_set(a), a)
}

/// A sequence is eventually in each of its tails.
theorem sequence_tail_set_eventually_contains_sequence(a: Nat -> Real, m: Nat) {
    seq_eventually_in_real_set(sequence_tail_set(a, m), a)
} by {
    forall(n: Nat) {
        if m <= n {
            sequence_tail_set_contains_term(a, m, n)
            sequence_tail_set(a, m).contains(a(n))
        }
    }
    exists(n0: Nat) {
        forall(n: Nat) {
            n0 <= n implies sequence_tail_set(a, m).contains(a(n))
        }
    }
}

/// A sequence is frequently in each of its tails.
theorem sequence_tail_set_frequently_contains_sequence(a: Nat -> Real, m: Nat) {
    seq_frequently_in_real_set(sequence_tail_set(a, m), a)
} by {
    sequence_tail_set_eventually_contains_sequence(a, m)
    seq_eventually_in_real_set(sequence_tail_set(a, m), a)
    from real.sequence_set_membership import seq_eventually_in_real_set_frequently
    seq_eventually_in_real_set_frequently(sequence_tail_set(a, m), a)
    seq_frequently_in_real_set(sequence_tail_set(a, m), a)
}

/// Pointwise sequence membership is equivalent to range containment.
theorem seq_in_real_set_iff_range_subset(s: Set[Real], a: Nat -> Real) {
    seq_in_real_set(s, a) = sequence_range_set(a).subset(s)
} by {
    if seq_in_real_set(s, a) {
        forall(x: Real) {
            if sequence_range_set(a).contains(x) {
                let n: Nat satisfy {
                    x = a(n)
                }
                s.contains(a(n))
                s.contains(x)
            }
        }
        sequence_range_set(a).subset(s)
    }
    if sequence_range_set(a).subset(s) {
        forall(n: Nat) {
            sequence_range_set_contains_term(a, n)
            sequence_range_set(a).contains(a(n))
            subset_contains(sequence_range_set(a), s, a(n))
            s.contains(a(n))
        }
        seq_in_real_set(s, a)
    }
    seq_in_real_set(s, a) = sequence_range_set(a).subset(s)
}

/// Eventual sequence membership is equivalent to containment of some tail.
theorem seq_eventually_in_real_set_iff_tail_subset(s: Set[Real], a: Nat -> Real) {
    seq_eventually_in_real_set(s, a) = exists(m: Nat) { sequence_tail_set(a, m).subset(s) }
} by {
    if seq_eventually_in_real_set(s, a) {
        let m: Nat satisfy {
            forall(n: Nat) {
                m <= n implies s.contains(a(n))
            }
        }
        forall(x: Real) {
            if sequence_tail_set(a, m).contains(x) {
                let n: Nat satisfy {
                    m <= n and x = a(n)
                }
                s.contains(a(n))
                s.contains(x)
            }
        }
        sequence_tail_set(a, m).subset(s)
        exists(k: Nat) {
            sequence_tail_set(a, k).subset(s)
        }
    }
    if exists(m: Nat) { sequence_tail_set(a, m).subset(s) } {
        let m: Nat satisfy {
            sequence_tail_set(a, m).subset(s)
        }
        forall(n: Nat) {
            if m <= n {
                sequence_tail_set_contains_term(a, m, n)
                sequence_tail_set(a, m).contains(a(n))
                subset_contains(sequence_tail_set(a, m), s, a(n))
                s.contains(a(n))
            }
        }
        exists(k: Nat) {
            forall(n: Nat) {
                k <= n implies s.contains(a(n))
            }
        }
        seq_eventually_in_real_set(s, a)
    }
    seq_eventually_in_real_set(s, a) = exists(m: Nat) { sequence_tail_set(a, m).subset(s) }
}

/// Frequent sequence membership is equivalent to every tail meeting the set.
theorem seq_frequently_in_real_set_iff_tail_meets(s: Set[Real], a: Nat -> Real) {
    seq_frequently_in_real_set(s, a) = forall(m: Nat) { sequence_tail_meets_set(a, m, s) }
} by {
    if seq_frequently_in_real_set(s, a) {
        forall(m: Nat) {
            let n: Nat satisfy {
                m <= n and s.contains(a(n))
            }
            sequence_tail_set_contains_term(a, m, n)
            sequence_tail_set(a, m).contains(a(n))
            exists(x: Real) {
                sequence_tail_set(a, m).contains(x) and s.contains(x)
            }
            sequence_tail_meets_set(a, m, s)
        }
    }
    if forall(m: Nat) { sequence_tail_meets_set(a, m, s) } {
        forall(m: Nat) {
            sequence_tail_meets_set(a, m, s)
            let x: Real satisfy {
                sequence_tail_set(a, m).contains(x) and s.contains(x)
            }
            let n: Nat satisfy {
                m <= n and x = a(n)
            }
            s.contains(a(n))
            exists(k: Nat) {
                m <= k and s.contains(a(k))
            }
        }
        seq_frequently_in_real_set(s, a)
    }
    seq_frequently_in_real_set(s, a) = forall(m: Nat) { sequence_tail_meets_set(a, m, s) }
}

/// A tail meeting a set also makes every earlier tail meet that set.
theorem sequence_tail_meets_set_of_le(a: Nat -> Real, m: Nat, n: Nat, s: Set[Real]) {
    m <= n and sequence_tail_meets_set(a, n, s) implies sequence_tail_meets_set(a, m, s)
} by {
    if m <= n and sequence_tail_meets_set(a, n, s) {
        let x: Real satisfy {
            sequence_tail_set(a, n).contains(x) and s.contains(x)
        }
        sequence_tail_set_contains_of_le(a, m, n, x)
        sequence_tail_set(a, m).contains(x)
        exists(y: Real) {
            sequence_tail_set(a, m).contains(y) and s.contains(y)
        }
        sequence_tail_meets_set(a, m, s)
    }
}

/// If a tail is contained in a set, every later tail is contained in that set.
theorem sequence_tail_subset_of_tail_subset(a: Nat -> Real, m: Nat, n: Nat, s: Set[Real]) {
    m <= n and sequence_tail_set(a, m).subset(s) implies sequence_tail_set(a, n).subset(s)
} by {
    if m <= n and sequence_tail_set(a, m).subset(s) {
        sequence_tail_set_subset_of_le(a, m, n)
        sequence_tail_set(a, n).subset(sequence_tail_set(a, m))
        subset_trans[Real](sequence_tail_set(a, n), sequence_tail_set(a, m), s)
        sequence_tail_set(a, n).subset(s)
    }
}

/// Intersecting a tail with a set records a tail meeting witness.
theorem sequence_tail_meets_set_intersection_contains(a: Nat -> Real, m: Nat, s: Set[Real], x: Real) {
    sequence_tail_set(a, m).contains(x) and s.contains(x)
    implies sequence_tail_set(a, m).intersection(s).contains(x)
} by {
    if sequence_tail_set(a, m).contains(x) and s.contains(x) {
        intersection_contains_intro(sequence_tail_set(a, m), s, x)
    }
}

/// A tail meets a set exactly when its intersection with that set is inhabited.
theorem sequence_tail_meets_set_iff_intersection_inhabited(a: Nat -> Real, m: Nat, s: Set[Real]) {
    sequence_tail_meets_set(a, m, s) = exists(x: Real) {
        sequence_tail_set(a, m).intersection(s).contains(x)
    }
} by {
    if sequence_tail_meets_set(a, m, s) {
        let x: Real satisfy {
            sequence_tail_set(a, m).contains(x) and s.contains(x)
        }
        sequence_tail_meets_set_intersection_contains(a, m, s, x)
        sequence_tail_set(a, m).intersection(s).contains(x)
        exists(y: Real) {
            sequence_tail_set(a, m).intersection(s).contains(y)
        }
    }
    if exists(x: Real) { sequence_tail_set(a, m).intersection(s).contains(x) } {
        let x: Real satisfy {
            sequence_tail_set(a, m).intersection(s).contains(x)
        }
        sequence_tail_set(a, m).contains(x)
        s.contains(x)
        exists(y: Real) {
            sequence_tail_set(a, m).contains(y) and s.contains(y)
        }
        sequence_tail_meets_set(a, m, s)
    }
    sequence_tail_meets_set(a, m, s) = exists(x: Real) {
        sequence_tail_set(a, m).intersection(s).contains(x)
    }
}
