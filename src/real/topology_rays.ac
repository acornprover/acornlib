from data.basic.set import Set, compl_contains_eq, set_ext
from real.real_base import close_imp_bounds
from real.real_field import Real
from real.real_seq import eps_lt_half, lt_imp_minus_pos
from real.topology import interior_point_intro, is_closed_set, is_interior_point, is_open_set
from real.topology_closed_open import complement_of_open_is_closed

/// True if a real number lies strictly above a lower endpoint.
define open_upper_ray_contains(lower: Real, x: Real) -> Bool {
    lower < x
}

/// True if a real number lies strictly below an upper endpoint.
define open_lower_ray_contains(upper: Real, x: Real) -> Bool {
    x < upper
}

/// True if a real number lies above or at a lower endpoint.
define closed_upper_ray_contains(lower: Real, x: Real) -> Bool {
    lower <= x
}

/// True if a real number lies below or at an upper endpoint.
define closed_lower_ray_contains(upper: Real, x: Real) -> Bool {
    x <= upper
}

/// The open ray `(lower, +infinity)` as a set of real numbers.
define open_upper_ray(lower: Real) -> Set[Real] {
    Set[Real].new(open_upper_ray_contains(lower))
}

/// The open ray `(-infinity, upper)` as a set of real numbers.
define open_lower_ray(upper: Real) -> Set[Real] {
    Set[Real].new(open_lower_ray_contains(upper))
}

/// The closed ray `[lower, +infinity)` as a set of real numbers.
define closed_upper_ray(lower: Real) -> Set[Real] {
    Set[Real].new(closed_upper_ray_contains(lower))
}

/// The closed ray `(-infinity, upper]` as a set of real numbers.
define closed_lower_ray(upper: Real) -> Set[Real] {
    Set[Real].new(closed_lower_ray_contains(upper))
}

/// Membership in an open upper ray is strict inequality above the endpoint.
theorem open_upper_ray_contains_eq(lower: Real, x: Real) {
    open_upper_ray(lower).contains(x) = (lower < x)
}

/// Membership in an open lower ray is strict inequality below the endpoint.
theorem open_lower_ray_contains_eq(upper: Real, x: Real) {
    open_lower_ray(upper).contains(x) = (x < upper)
}

/// Membership in a closed upper ray is inequality above the endpoint.
theorem closed_upper_ray_contains_eq(lower: Real, x: Real) {
    closed_upper_ray(lower).contains(x) = (lower <= x)
}

/// Membership in a closed lower ray is inequality below the endpoint.
theorem closed_lower_ray_contains_eq(upper: Real, x: Real) {
    closed_lower_ray(upper).contains(x) = (x <= upper)
}

/// A positive summand is below any upper bound for twice that summand.
theorem positive_half_lt_bound(eps: Real, bound: Real) {
    eps.is_positive and eps + eps < bound implies eps < bound
} by {
    if eps.is_positive and eps + eps < bound {
        Real.0 < eps
        eps < eps + eps
        eps < bound
    }
}

/// Open upper rays are open real sets.
theorem open_upper_ray_is_open(lower: Real) {
    is_open_set(open_upper_ray(lower))
} by {
    forall(x: Real) {
        if open_upper_ray(lower).contains(x) {
            open_upper_ray_contains_eq(lower, x)
            lower < x
            lt_imp_minus_pos(lower, x)
            let gap = x - lower
            gap.is_positive
            eps_lt_half(gap)
            let eps: Real satisfy {
                eps.is_positive and eps + eps < gap
            }
            eps.is_positive
            positive_half_lt_bound(eps, gap)
            eps < gap
            eps < x - lower
            lower + eps < x
            lower < x - eps
            forall(y: Real) {
                if y.is_close(x, eps) {
                    close_imp_bounds(y, x, eps)
                    x - eps < y
                    lower < y
                    open_upper_ray_contains_eq(lower, y)
                    open_upper_ray(lower).contains(y)
                }
            }
            interior_point_intro(open_upper_ray(lower), x, eps)
            is_interior_point(open_upper_ray(lower), x)
        }
    }
    is_open_set(open_upper_ray(lower))
}

/// Open lower rays are open real sets.
theorem open_lower_ray_is_open(upper: Real) {
    is_open_set(open_lower_ray(upper))
} by {
    forall(x: Real) {
        if open_lower_ray(upper).contains(x) {
            open_lower_ray_contains_eq(upper, x)
            x < upper
            lt_imp_minus_pos(x, upper)
            let gap = upper - x
            gap.is_positive
            eps_lt_half(gap)
            let eps: Real satisfy {
                eps.is_positive and eps + eps < gap
            }
            eps.is_positive
            positive_half_lt_bound(eps, gap)
            eps < gap
            eps < upper - x
            x + eps < upper
            forall(y: Real) {
                if y.is_close(x, eps) {
                    close_imp_bounds(y, x, eps)
                    y < x + eps
                    y < upper
                    open_lower_ray_contains_eq(upper, y)
                    open_lower_ray(upper).contains(y)
                }
            }
            interior_point_intro(open_lower_ray(upper), x, eps)
            is_interior_point(open_lower_ray(upper), x)
        }
    }
    is_open_set(open_lower_ray(upper))
}

/// A closed upper ray is the complement of the corresponding open lower ray.
theorem closed_upper_ray_eq_complement_open_lower_ray(lower: Real) {
    closed_upper_ray(lower) = open_lower_ray(lower).c
} by {
    forall(x: Real) {
        if closed_upper_ray(lower).contains(x) {
            closed_upper_ray_contains_eq(lower, x)
            lower <= x
            not x < lower
            open_lower_ray_contains_eq(lower, x)
            not open_lower_ray(lower).contains(x)
            compl_contains_eq(open_lower_ray(lower), x)
            open_lower_ray(lower).c.contains(x)
        }
        if open_lower_ray(lower).c.contains(x) {
            compl_contains_eq(open_lower_ray(lower), x)
            not open_lower_ray(lower).contains(x)
            open_lower_ray_contains_eq(lower, x)
            not x < lower
            lower <= x
            closed_upper_ray_contains_eq(lower, x)
            closed_upper_ray(lower).contains(x)
        }
        closed_upper_ray(lower).contains(x) = open_lower_ray(lower).c.contains(x)
    }
    set_ext(closed_upper_ray(lower), open_lower_ray(lower).c)
}

/// A closed lower ray is the complement of the corresponding open upper ray.
theorem closed_lower_ray_eq_complement_open_upper_ray(upper: Real) {
    closed_lower_ray(upper) = open_upper_ray(upper).c
} by {
    forall(x: Real) {
        if closed_lower_ray(upper).contains(x) {
            closed_lower_ray_contains_eq(upper, x)
            x <= upper
            not upper < x
            open_upper_ray_contains_eq(upper, x)
            not open_upper_ray(upper).contains(x)
            compl_contains_eq(open_upper_ray(upper), x)
            open_upper_ray(upper).c.contains(x)
        }
        if open_upper_ray(upper).c.contains(x) {
            compl_contains_eq(open_upper_ray(upper), x)
            not open_upper_ray(upper).contains(x)
            open_upper_ray_contains_eq(upper, x)
            not upper < x
            x <= upper
            closed_lower_ray_contains_eq(upper, x)
            closed_lower_ray(upper).contains(x)
        }
        closed_lower_ray(upper).contains(x) = open_upper_ray(upper).c.contains(x)
    }
    set_ext(closed_lower_ray(upper), open_upper_ray(upper).c)
}

/// Closed upper rays are closed real sets.
theorem closed_upper_ray_is_closed(lower: Real) {
    is_closed_set(closed_upper_ray(lower))
} by {
    open_lower_ray_is_open(lower)
    complement_of_open_is_closed(open_lower_ray(lower))
    is_closed_set(open_lower_ray(lower).c)
    closed_upper_ray_eq_complement_open_lower_ray(lower)
    is_closed_set(closed_upper_ray(lower))
}

/// Closed lower rays are closed real sets.
theorem closed_lower_ray_is_closed(upper: Real) {
    is_closed_set(closed_lower_ray(upper))
} by {
    open_upper_ray_is_open(upper)
    complement_of_open_is_closed(open_upper_ray(upper))
    is_closed_set(open_upper_ray(upper).c)
    closed_lower_ray_eq_complement_open_upper_ray(upper)
    is_closed_set(closed_lower_ray(upper))
}
