from data.basic.functions import identity_fn
from order import lt_imp_ne_symm, not_lt_self
from real.real_base import Real, add_comm, close_comm, lt_add_pos, pos_imp_eq_abs, self_close,
    sub_cancels
from real.real_field import mul_left_cancel
from real.real_ring import mul_one_right, mul_zero_left
from real.real_seq import close_and_lt_imp_close, eps_lt_half, eps_smaller_than_both,
    is_close_triangle, neq_imp_abs_diff_pos, smaller_pos, sub_zero_imp_eq

/// A universal proposition holds at each element.
theorem forall_elim[T](p: T -> Bool, x: T) {
    (forall(y: T) { p(y) }) implies p(x)
} by {
    if forall(y: T) { p(y) } {
        p(x)
    }
}

/// The difference quotient of a real-valued function at a base point.
define difference_quotient(f: Real -> Real, x0: Real, x: Real) -> Real {
    (f(x) - f(x0)) / (x - x0)
}

/// True if a real-valued function has derivative `d` at a point.
define has_derivative_at(f: Real -> Real, x0: Real, d: Real) -> Bool {
    forall(eps: Real) {
        eps.is_positive implies exists(delta: Real) {
            delta.is_positive and forall(x: Real) {
                x != x0 and x.is_close(x0, delta)
                implies difference_quotient(f, x0, x).is_close(d, eps)
            }
        }
    }
}

/// True if a real-valued function is differentiable at a point.
define differentiable_at(f: Real -> Real, x0: Real) -> Bool {
    exists(d: Real) {
        has_derivative_at(f, x0, d)
    }
}

/// Distinct real points have nonzero difference.
theorem sub_ne_zero_of_ne(x: Real, x0: Real) {
    x != x0 implies x - x0 != Real.0
} by {
    if x != x0 {
        if x - x0 = Real.0 {
            sub_zero_imp_eq(x, x0)
            x = x0
            false
        }
    }
}

/// A positive radius contains a point distinct from the center.
theorem exists_punctured_close_point(x0: Real, delta: Real) {
    delta.is_positive implies exists(x: Real) {
        x != x0 and x.is_close(x0, delta)
    }
} by {
    if delta.is_positive {
        smaller_pos(delta)
        let h: Real satisfy {
            h.is_positive and h < delta
        }
        let x = x0 + h
        lt_add_pos(x0, h)
        x0 < x
        lt_imp_ne_symm(x0, x)
        x != x0
        x - x0 = x0 + h - x0
        add_comm(x0, h)
        x0 + h = h + x0
        x0 + h - x0 = h + x0 - x0
        sub_cancels(h, x0)
        h + x0 - x0 = h
        x - x0 = h
        pos_imp_eq_abs(h)
        h = h.abs
        (x - x0).abs = h
        (x - x0).abs < delta
        x.is_close(x0, delta)
        exists(y: Real) {
            y != x0 and y.is_close(x0, delta)
        }
    }
}

/// Two derivative candidates close to the same difference quotient are close to each other.
theorem derivative_values_close_from_same_witness(
    q: Real, d1: Real, d2: Real, eps: Real
) {
    q.is_close(d1, eps) and q.is_close(d2, eps) implies d1.is_close(d2, eps + eps)
} by {
    if q.is_close(d1, eps) and q.is_close(d2, eps) {
        close_comm(q, d1, eps)
        d1.is_close(q, eps)
        close_comm(q, d2, eps)
        d2.is_close(q, eps)
        is_close_triangle(d1, d2, q, eps, eps)
        d1.is_close(d2, eps + eps)
    }
}

/// A derivative at a point gives a punctured-neighborhood estimate for every positive tolerance.
theorem has_derivative_at_delta(f: Real -> Real, x0: Real, d: Real, eps: Real) {
    has_derivative_at(f, x0, d) and eps.is_positive implies exists(delta: Real) {
        delta.is_positive and forall(x: Real) {
            x != x0 and x.is_close(x0, delta)
            implies difference_quotient(f, x0, x).is_close(d, eps)
        }
    }
} by {
    has_derivative_at(f, x0, d) = forall(e: Real) {
        e.is_positive implies exists(delta: Real) {
            delta.is_positive and forall(x: Real) {
                x != x0 and x.is_close(x0, delta)
                implies difference_quotient(f, x0, x).is_close(d, e)
            }
        }
    }
    if has_derivative_at(f, x0, d) and eps.is_positive {
        exists(delta2: Real) {
            delta2.is_positive and forall(x: Real) {
                x != x0 and x.is_close(x0, delta2)
                implies difference_quotient(f, x0, x).is_close(d, eps)
            }
        }
    }
}

/// The difference quotient of a constant real function is zero away from the base point.
theorem difference_quotient_constant_eq_zero(c: Real, x0: Real, x: Real) {
    x != x0 implies difference_quotient(constant[Real, Real](c), x0, x) = Real.0
} by {
    if x != x0 {
        constant[Real, Real](c, x) = c
        constant[Real, Real](c, x0) = c
        constant[Real, Real](c, x) - constant[Real, Real](c, x0) = c - c
        c - c = Real.0
        Real.0 / (x - x0) = Real.0 * (x - x0).inverse
        mul_zero_left((x - x0).inverse)
        Real.0 / (x - x0) = Real.0
        difference_quotient(constant[Real, Real](c), x0, x) = Real.0
    }
}

/// The difference quotient of the identity function is one away from the base point.
theorem difference_quotient_identity_eq_one(x0: Real, x: Real) {
    x != x0 implies difference_quotient(identity_fn[Real], x0, x) = Real.1
} by {
    if x != x0 {
        sub_ne_zero_of_ne(x, x0)
        x - x0 != Real.0
        identity_fn[Real](x) = x
        identity_fn[Real](x0) = x0
        identity_fn[Real](x) - identity_fn[Real](x0) = x - x0
        (x - x0) * Real.1 = x - x0
        mul_one_right(x - x0)
        mul_left_cancel(x - x0, x - x0, Real.1)
        (x - x0) / (x - x0) = Real.1
        difference_quotient(identity_fn[Real], x0, x) = Real.1
    }
}

/// Constant real functions have derivative zero at every point.
theorem constant_has_derivative_at(c: Real, x0: Real) {
    has_derivative_at(constant[Real, Real](c), x0, Real.0)
} by {
    forall(eps: Real) {
        if eps.is_positive {
            forall(x: Real) {
                if x != x0 and x.is_close(x0, eps) {
                    difference_quotient_constant_eq_zero(c, x0, x)
                    difference_quotient(constant[Real, Real](c), x0, x) = Real.0
                    self_close(Real.0, eps)
                    difference_quotient(constant[Real, Real](c), x0, x).is_close(Real.0, eps)
                }
            }
            exists(delta: Real) {
                delta.is_positive and forall(x: Real) {
                    x != x0 and x.is_close(x0, delta)
                    implies difference_quotient(constant[Real, Real](c), x0, x).is_close(Real.0, eps)
                }
            }
        }
    }
    has_derivative_at(constant[Real, Real](c), x0, Real.0) = forall(eps: Real) {
        eps.is_positive implies exists(delta: Real) {
            delta.is_positive and forall(x: Real) {
                x != x0 and x.is_close(x0, delta)
                implies difference_quotient(constant[Real, Real](c), x0, x).is_close(Real.0, eps)
            }
        }
    }
    if not has_derivative_at(constant[Real, Real](c), x0, Real.0) {
        not forall(eps: Real) {
            eps.is_positive implies exists(delta: Real) {
                delta.is_positive and forall(x: Real) {
                    x != x0 and x.is_close(x0, delta)
                    implies difference_quotient(constant[Real, Real](c), x0, x).is_close(Real.0, eps)
                }
            }
        }
        let bad_eps: Real satisfy {
            bad_eps.is_positive and forall(delta: Real) {
                not (delta.is_positive and forall(x: Real) {
                    x != x0 and x.is_close(x0, delta)
                    implies difference_quotient(constant[Real, Real](c), x0, x).is_close(Real.0, bad_eps)
                })
            }
        }
        exists(delta: Real) {
            delta.is_positive and forall(x: Real) {
                x != x0 and x.is_close(x0, delta)
                implies difference_quotient(constant[Real, Real](c), x0, x).is_close(Real.0, bad_eps)
            }
        }
        false
    }
}

/// The identity function has derivative one at every point.
theorem identity_has_derivative_at(x0: Real) {
    has_derivative_at(identity_fn[Real], x0, Real.1)
} by {
    forall(eps: Real) {
        if eps.is_positive {
            forall(x: Real) {
                if x != x0 and x.is_close(x0, eps) {
                    difference_quotient_identity_eq_one(x0, x)
                    difference_quotient(identity_fn[Real], x0, x) = Real.1
                    self_close(Real.1, eps)
                    difference_quotient(identity_fn[Real], x0, x).is_close(Real.1, eps)
                }
            }
            exists(delta: Real) {
                delta.is_positive and forall(x: Real) {
                    x != x0 and x.is_close(x0, delta)
                    implies difference_quotient(identity_fn[Real], x0, x).is_close(Real.1, eps)
                }
            }
        }
    }
    has_derivative_at(identity_fn[Real], x0, Real.1) = forall(eps: Real) {
        eps.is_positive implies exists(delta: Real) {
            delta.is_positive and forall(x: Real) {
                x != x0 and x.is_close(x0, delta)
                implies difference_quotient(identity_fn[Real], x0, x).is_close(Real.1, eps)
            }
        }
    }
    if not has_derivative_at(identity_fn[Real], x0, Real.1) {
        not forall(eps: Real) {
            eps.is_positive implies exists(delta: Real) {
                delta.is_positive and forall(x: Real) {
                    x != x0 and x.is_close(x0, delta)
                    implies difference_quotient(identity_fn[Real], x0, x).is_close(Real.1, eps)
                }
            }
        }
        let bad_eps: Real satisfy {
            bad_eps.is_positive and forall(delta: Real) {
                not (delta.is_positive and forall(x: Real) {
                    x != x0 and x.is_close(x0, delta)
                    implies difference_quotient(identity_fn[Real], x0, x).is_close(Real.1, bad_eps)
                })
            }
        }
        exists(delta: Real) {
            delta.is_positive and forall(x: Real) {
                x != x0 and x.is_close(x0, delta)
                implies difference_quotient(identity_fn[Real], x0, x).is_close(Real.1, bad_eps)
            }
        }
        false
    }
}

/// Two derivative values for the same real function at the same point are equal.
theorem has_derivative_at_unique(f: Real -> Real, x0: Real, d1: Real, d2: Real) {
    has_derivative_at(f, x0, d1) and has_derivative_at(f, x0, d2) implies d1 = d2
} by {
    if d1 != d2 {
        neq_imp_abs_diff_pos(d1, d2)
        let gap = (d1 - d2).abs
        gap.is_positive
        eps_lt_half(gap)
        let eps: Real satisfy {
            eps.is_positive and eps + eps < gap
        }

        has_derivative_at_delta(f, x0, d1, eps)
        let delta1: Real satisfy {
            delta1.is_positive and forall(x: Real) {
                x != x0 and x.is_close(x0, delta1)
                implies difference_quotient(f, x0, x).is_close(d1, eps)
            }
        }
        has_derivative_at_delta(f, x0, d2, eps)
        let delta2: Real satisfy {
            delta2.is_positive and forall(x: Real) {
                x != x0 and x.is_close(x0, delta2)
                implies difference_quotient(f, x0, x).is_close(d2, eps)
            }
        }
        eps_smaller_than_both(delta1, delta2)
        let delta: Real satisfy {
            delta.is_positive and delta < delta1 and delta < delta2
        }
        exists_punctured_close_point(x0, delta)
        let x: Real satisfy {
            x != x0 and x.is_close(x0, delta)
        }
        close_and_lt_imp_close(x, x0, delta, delta1)
        x.is_close(x0, delta1)
        close_and_lt_imp_close(x, x0, delta, delta2)
        x.is_close(x0, delta2)
        x != x0 and x.is_close(x0, delta1)
        forall_elim[Real](function(y: Real) {
            y != x0 and y.is_close(x0, delta1) implies
                difference_quotient(f, x0, y).is_close(d1, eps)
        }, x)
        function(y: Real) {
            y != x0 and y.is_close(x0, delta1) implies
                difference_quotient(f, x0, y).is_close(d1, eps)
        }(x)
        difference_quotient(f, x0, x).is_close(d1, eps)
        x != x0 and x.is_close(x0, delta2)
        forall_elim[Real](function(y: Real) {
            y != x0 and y.is_close(x0, delta2) implies
                difference_quotient(f, x0, y).is_close(d2, eps)
        }, x)
        function(y: Real) {
            y != x0 and y.is_close(x0, delta2) implies
                difference_quotient(f, x0, y).is_close(d2, eps)
        }(x)
        difference_quotient(f, x0, x).is_close(d2, eps)
        derivative_values_close_from_same_witness(difference_quotient(f, x0, x), d1, d2, eps)
        d1.is_close(d2, eps + eps)
        d1.is_close(d2, eps + eps) and eps + eps < gap
        close_and_lt_imp_close(d1, d2, eps + eps, gap)
        d1.is_close(d2, gap)
        d1.is_close(d2, gap) = (d1 - d2).abs < gap
        (d1 - d2).abs < gap
        (d1 - d2).abs = gap
        gap < gap
        not_lt_self(gap)
        false
    }
}

/// Constant real functions are differentiable at every point.
theorem constant_differentiable_at(c: Real, x0: Real) {
    differentiable_at(constant[Real, Real](c), x0)
} by {
    constant_has_derivative_at(c, x0)
    exists(d: Real) {
        has_derivative_at(constant[Real, Real](c), x0, d)
    }
}

/// The identity function is differentiable at every point.
theorem identity_differentiable_at(x0: Real) {
    differentiable_at(identity_fn[Real], x0)
} by {
    identity_has_derivative_at(x0)
    exists(d: Real) {
        has_derivative_at(identity_fn[Real], x0, d)
    }
}
