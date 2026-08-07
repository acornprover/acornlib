from data.basic.functions import compose, identity_fn
from nat import Nat
from real.continuity_base import Real, continuous, continuous_at, continuous_condition, uniform,
    uniform_condition, uniform_imp_continuous, uniform_witness
from real.real_seq import converges_to, tail_bound, tail_bound_implies_is_close, cauchy_bound,
    converges
from real.cauchy_criterion import is_cauchy_seq, cauchy_bound_indices,
    cauchy_imp_converges, converges_imp_cauchy_seq, converges_imp_cauchy_bound

/// The identity function on the reals is uniformly continuous.
theorem identity_function_is_uniform {
    uniform(identity_fn[Real])
} by {
    forall(eps: Real) {
        if eps.is_positive {
            forall(x: Real, y: Real) {
                if x.is_close(y, eps) {
                    identity_fn[Real](x) = x
                    identity_fn[Real](y) = y
                    identity_fn[Real](x).is_close(identity_fn[Real](y), eps)
                }
            }
            uniform_condition(identity_fn[Real], eps, eps)
            eps.is_positive and uniform_condition(identity_fn[Real], eps, eps)
            exists(delta: Real) {
                delta.is_positive and uniform_condition(identity_fn[Real], delta, eps)
            }
        }
    }
    uniform(identity_fn[Real]) = forall(eps2: Real) {
        eps2.is_positive implies exists(delta2: Real) {
            delta2.is_positive and uniform_condition(identity_fn[Real], delta2, eps2)
        }
    }
    uniform(identity_fn[Real])
}

/// The identity function on the reals is continuous.
theorem identity_function_is_continuous {
    continuous(identity_fn[Real])
} by {
    identity_function_is_uniform
    uniform_imp_continuous(identity_fn[Real])
    continuous(identity_fn[Real])
}

/// A function continuous at a point preserves convergence to that point.
theorem continuous_at_preserves_sequence_limit(f: Real -> Real, a: Nat -> Real, x: Real) {
    continuous_at(f, x) and converges_to(a, x) implies converges_to(compose(f, a), f(x))
} by {
    continuous_at(f, x) = forall(eps2: Real) {
        eps2.is_positive implies exists(delta: Real) {
            delta.is_positive and continuous_condition(f, x, delta, eps2)
        }
    }
    converges_to(a, x) = forall(eps2: Real) {
        eps2.is_positive implies exists(n: Nat) {
            tail_bound(a, x, n, eps2)
        }
    }
    forall(eps: Real) {
        if eps.is_positive {
            let delta: Real satisfy {
                delta.is_positive and continuous_condition(f, x, delta, eps)
            }
            let n: Nat satisfy {
                tail_bound(a, x, n, delta)
            }
            continuous_condition(f, x, delta, eps) = forall(y: Real) {
                y.is_close(x, delta) implies f(y).is_close(f(x), eps)
            }
            forall(i: Nat) {
                if n <= i {
                    tail_bound_implies_is_close(a, x, n, delta, i)
                    a(i).is_close(x, delta)
                    f(a(i)).is_close(f(x), eps)
                    compose(f, a, i) = f(a(i))
                    compose(f, a, i).is_close(f(x), eps)
                }
            }
            tail_bound(compose(f, a), f(x), n, eps)
        }
    }
    converges_to(compose(f, a), f(x)) = forall(eps2: Real) {
        eps2.is_positive implies exists(n2: Nat) {
            tail_bound(compose(f, a), f(x), n2, eps2)
        }
    }
}

/// A continuous real function preserves sequence limits.
theorem continuous_function_preserves_sequence_limit(f: Real -> Real, a: Nat -> Real, x: Real) {
    continuous(f) and converges_to(a, x) implies converges_to(compose(f, a), f(x))
} by {
    continuous(f) = forall(y: Real) {
        continuous_at(f, y)
    }
    continuous_at(f, x)
    continuous_at_preserves_sequence_limit(f, a, x)
    converges_to(compose(f, a), f(x))
}

/// A uniform-continuity modulus carries one close pair to a close image pair.
theorem uniform_condition_maps_close(f: Real -> Real, x: Real, y: Real, delta: Real, eps: Real) {
    uniform_condition(f, delta, eps) and x.is_close(y, delta) implies f(x).is_close(f(y), eps)
} by {
    if uniform_condition(f, delta, eps) and x.is_close(y, delta) {
        if not f(x).is_close(f(y), eps) {
            exists(r1: Real, r2: Real) {
                r1.is_close(r2, delta) and not f(r1).is_close(f(r2), eps)
            }
            uniform_condition(f, delta, eps) = forall(r1: Real, r2: Real) {
                r1.is_close(r2, delta) implies f(r1).is_close(f(r2), eps)
            }
            not uniform_condition(f, delta, eps)
            false
        }
        f(x).is_close(f(y), eps)
    }
}

/// A uniform-continuity modulus transports a Cauchy tail bound through composition.
theorem uniform_condition_preserves_cauchy_bound(f: Real -> Real, a: Nat -> Real,
    n: Nat, delta: Real, eps: Real) {
    uniform_condition(f, delta, eps) and cauchy_bound(a, n, delta)
        implies cauchy_bound(compose(f, a), n, eps)
} by {
    if uniform_condition(f, delta, eps) and cauchy_bound(a, n, delta) {
        forall(i: Nat, j: Nat) {
            if n <= i and n <= j {
                cauchy_bound_indices(a, n, delta, i, j)
                a(i).is_close(a(j), delta)
                uniform_condition_maps_close(f, a(i), a(j), delta, eps)
                f(a(i)).is_close(f(a(j)), eps)
                compose(f, a, i) = f(a(i))
                compose(f, a, j) = f(a(j))
                compose(f, a, i).is_close(compose(f, a, j), eps)
            }
        }
        if not cauchy_bound(compose(f, a), n, eps) {
            cauchy_bound(compose(f, a), n, eps) = forall(i: Nat, j: Nat) {
                n <= i and n <= j implies compose(f, a, i).is_close(compose(f, a, j), eps)
            }
            exists(i: Nat, j: Nat) {
                n <= i and n <= j and not compose(f, a, i).is_close(compose(f, a, j), eps)
            }
            let i: Nat satisfy {
                exists(j: Nat) {
                    n <= i and n <= j and not compose(f, a, i).is_close(compose(f, a, j), eps)
                }
            }
            let j: Nat satisfy {
                n <= i and n <= j and not compose(f, a, i).is_close(compose(f, a, j), eps)
            }
            n <= i and n <= j implies compose(f, a, i).is_close(compose(f, a, j), eps)
            compose(f, a, i).is_close(compose(f, a, j), eps)
            false
        }
        cauchy_bound(compose(f, a), n, eps)
    }
}

/// Uniform continuity transports the existing Cauchy-bound convergence predicate.
theorem uniform_preserves_converges(f: Real -> Real, a: Nat -> Real) {
    uniform(f) and converges(a) implies converges(compose(f, a))
} by {
    let fa = compose(f, a)
    forall(eps: Real) {
        if eps.is_positive {
            uniform(f)
            uniform_witness(f, eps)
            let delta: Real satisfy {
                delta.is_positive and uniform_condition(f, delta, eps)
            }
            converges(a)
            converges_imp_cauchy_bound(a, delta)
            let n: Nat satisfy {
                cauchy_bound(a, n, delta)
            }
            uniform_condition_preserves_cauchy_bound(f, a, n, delta, eps)
            cauchy_bound(compose(f, a), n, eps)
            compose(f, a) = fa
            cauchy_bound(fa, n, eps)
        }
    }
    if not converges(fa) {
        converges(fa) = forall(eps: Real) {
            eps.is_positive implies exists(n: Nat) {
                cauchy_bound(fa, n, eps)
            }
        }
        exists(eps: Real) {
            eps.is_positive and forall(n: Nat) {
                not cauchy_bound(fa, n, eps)
            }
        }
        let eps: Real satisfy {
            eps.is_positive and forall(n: Nat) {
                not cauchy_bound(fa, n, eps)
            }
        }
        eps.is_positive implies exists(n: Nat) {
            cauchy_bound(fa, n, eps)
        }
        let n: Nat satisfy {
            cauchy_bound(fa, n, eps)
        }
        not cauchy_bound(fa, n, eps)
        false
    }
    converges(fa)
    compose(f, a) = fa
    converges(compose(f, a))
}

/// Uniform continuity preserves Cauchy real sequences.
theorem uniform_preserves_cauchy_seq(f: Real -> Real, a: Nat -> Real) {
    uniform(f) and is_cauchy_seq(a) implies is_cauchy_seq(compose(f, a))
} by {
    let fa = compose(f, a)
    cauchy_imp_converges(a)
    converges(a)
    uniform_preserves_converges(f, a)
    converges(fa)
    converges_imp_cauchy_seq(fa)
    is_cauchy_seq(fa)
}
