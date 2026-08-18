from order import lt_of_lte_of_lt, lte_antisymm, lte_trans, not_lte_imp_gt,
    not_lt_imp_gte
from real.cauchy_schwarz import nonnegative_gap_imp_lte,
    real_square_add_expanded
from real.real_field import mul_left_cancel
from real.real_base import one_half_plus_one_half, one_half_positive
from real.real_ring import lte_mul_nonneg_left, lte_mul_nonneg_right,
    mul_neg_left, mul_neg_right, mul_nonneg, square_nonneg,
    square_zero_imp_zero
from real.log import rpow_neg_base
from real.sqrt import Real, sqrt_mul_self, sqrt_pos,
    sqrt_unique_nonneg

numerals Real

/// Multiplication by one half cancels a doubled real number.
theorem one_half_mul_double(x: Real) {
    Real.one_half * (x + x) = x
} by {
    Real.one_half * (x + x) = Real.one_half * x + Real.one_half * x
    Real.one_half * x + Real.one_half * x =
        (Real.one_half + Real.one_half) * x
    one_half_plus_one_half
    Real.one_half + Real.one_half = Real.1
    (Real.one_half + Real.one_half) * x = Real.1 * x
    Real.1 * x = x
}

/// One half is one divided by two.
theorem one_half_eq_one_div_two {
    Real.one_half = Real.1 / (Real.1 + Real.1)
} by {
    let two = Real.1 + Real.1
    Real.1 > Real.0
    two > Real.0
    two != Real.0
    two * Real.one_half = (Real.1 + Real.1) * Real.one_half
    (Real.1 + Real.1) * Real.one_half =
        Real.1 * Real.one_half + Real.1 * Real.one_half
    Real.1 * Real.one_half = Real.one_half
    two * Real.one_half = Real.one_half + Real.one_half
    one_half_plus_one_half
    Real.one_half + Real.one_half = Real.1
    two * Real.one_half = Real.1
    Real.1 = two * Real.one_half
    mul_left_cancel(Real.1, two, Real.one_half)
    Real.1 / two = Real.one_half
    Real.one_half = Real.1 / two
    two = Real.1 + Real.1
    Real.one_half = Real.1 / (Real.1 + Real.1)
}

/// A doubled upper bound gives the corresponding half upper bound.
theorem double_le_imp_le_half(x: Real, y: Real) {
    x + x <= y implies x <= Real.one_half * y
} by {
    one_half_positive
    Real.one_half >= Real.0
    lte_mul_nonneg_left(x + x, y, Real.one_half)
    Real.one_half * (x + x) <= Real.one_half * y
    one_half_mul_double(x)
    Real.one_half * (x + x) = x
    x <= Real.one_half * y
}

/// A nonnegative square gap gives the corresponding doubled-product inequality.
theorem square_gap_imp_double_le(q: Real, p: Real, r: Real) {
    q + -p + (-p + r) >= Real.0 implies p + p <= q + r
} by {
    q + -p + (-p + r) = (q + -p + r) + -p
    (q + -p + r) + -p >= Real.0
    nonnegative_gap_imp_lte(q + -p + r, p)
    p <= q + -p + r
    p + p <= (q + -p + r) + p
    (q + -p + r) + p = q + (-p + r) + p
    -p + r = r + -p
    q + (-p + r) + p = q + (r + -p) + p
    q + (r + -p) + p = q + r + -p + p
    q + r + -p + p = q + r + (-p + p)
    -p + p = Real.0
    q + r + (-p + p) = q + r + Real.0
    q + r + Real.0 = q + r
    (q + -p + r) + p = q + r
    p + p <= q + r
}

/// The doubled product of two reals is bounded by the sum of their squares.
theorem double_mul_le_square_sum(x: Real, y: Real) {
    x * y + x * y <= x * x + y * y
} by {
    square_nonneg(x - y)
    (x - y) * (x - y) >= Real.0
    x - y = x + -y
    real_square_add_expanded(x, -y)
    (x + -y) * (x + -y) = x * x + x * -y + (x * -y + -y * -y)
    mul_neg_right(x, y)
    x * -y = -(x * y)
    mul_neg_left(y, -y)
    -y * -y = -(y * -y)
    mul_neg_right(y, y)
    y * -y = -(y * y)
    -y * -y = y * y
    x * x + x * -y + (x * -y + -y * -y) =
        x * x + -(x * y) + (-(x * y) + y * y)
    x * x + -(x * y) + (-(x * y) + y * y) >= Real.0
    square_gap_imp_double_le(x * x, x * y, y * y)
    x * y + x * y <= x * x + y * y
}

/// Equality in the doubled-product bound forces the two factors to be equal.
theorem double_mul_eq_square_sum_imp_eq(x: Real, y: Real) {
    x * y + x * y = x * x + y * y implies x = y
} by {
    let p = x * y
    let q = x * x
    let r = y * y
    x - y = x + -y
    real_square_add_expanded(x, -y)
    (x + -y) * (x + -y) = x * x + x * -y + (x * -y + -y * -y)
    mul_neg_right(x, y)
    x * -y = -(x * y)
    mul_neg_left(y, -y)
    -y * -y = -(y * -y)
    mul_neg_right(y, y)
    y * -y = -(y * y)
    -y * -y = y * y
    x * x + x * -y + (x * -y + -y * -y) = q + -p + (-p + r)
    (x - y) * (x - y) = q + -p + (-p + r)
    p + p = q + r
    q + -p + (-p + r) = q + (-p + (-p + r))
    -p + (-p + r) = (-p + -p) + r
    (-p + -p) + r = r + (-p + -p)
    q + (-p + (-p + r)) = q + (r + (-p + -p))
    q + (r + (-p + -p)) = q + r + (-p + -p)
    q + -p + (-p + r) = q + r + (-p + -p)
    q + r + (-p + -p) = p + p + (-p + -p)
    p + p + (-p + -p) = p + (p + (-p + -p))
    p + (-p + -p) = (p + -p) + -p
    p + -p = Real.0
    (p + -p) + -p = Real.0 + -p
    Real.0 + -p = -p
    p + (-p + -p) = -p
    p + (p + (-p + -p)) = p + -p
    p + -p = Real.0
    q + -p + (-p + r) = Real.0
    (x - y) * (x - y) = Real.0
    square_zero_imp_zero(x - y)
    x - y = Real.0
    x + -y = Real.0
    x + -y + y = Real.0 + y
    x + (-y + y) = Real.0 + y
    -y + y = Real.0
    x + Real.0 = Real.0 + y
    x = y
}

/// Squaring is order-reflecting on nonnegative reals.
theorem square_le_square_of_nonneg(a: Real, b: Real) {
    a >= Real.0 and b >= Real.0 and a * a <= b * b implies a <= b
} by {
    if not a <= b {
        not_lte_imp_gt(a, b)
        a > b
        b <= a
        b >= Real.0
        not b.is_negative
        lte_mul_nonneg_left(b, a, b)
        b * b <= b * a
        a > Real.0
        a.is_positive
        b * a < a * a
        b * b < a * a
        a * a <= b * b
        false
    }
}

/// A returned square-root value is nonnegative for nonnegative inputs.
theorem sqrt_value_nonneg(x: Real, y: Real) {
    x.sqrt = Option.some(y) implies y >= Real.0
} by {
    if x > Real.0 {
        sqrt_pos(x)
        exists(z: Real) {
            x.sqrt = Option.some(z) and z > Real.0
        }
        let z: Real satisfy {
            x.sqrt = Option.some(z) and z > Real.0
        }
        Option.some(y) = Option.some(z)
        some_injective[Real](y, z)
        y = z
        y > Real.0
        y >= Real.0
    } else {
        if x < Real.0 {
            rpow_neg_base(x, Real.one_half)
            x.rpow(Real.one_half) = Option.none[Real]
            x.sqrt = Option.none[Real]
            x.sqrt = Option.some(y)
            Option.none[Real] = Option.some(y)
            false
        }
        not_lt_imp_gte(x, Real.0)
        x >= Real.0
        Real.0 <= x
        x <= Real.0
        lte_antisymm(Real.0, x)
        Real.0 = x
        x = Real.0
        x.sqrt = Option.some(Real.0)
        Option.some(y) = Option.some(Real.0)
        some_injective[Real](y, Real.0)
        y = Real.0
        y >= Real.0
    }
}

/// A returned square-root value squares to the original nonnegative input.
theorem sqrt_value_mul_self(x: Real, y: Real) {
    x >= Real.0 and x.sqrt = Option.some(y) implies y * y = x
} by {
    sqrt_mul_self(x)
    exists(z: Real) {
        x.sqrt = Option.some(z) and z * z = x
    }
    let z: Real satisfy {
        x.sqrt = Option.some(z) and z * z = x
    }
    Option.some(y) = Option.some(z)
    some_injective[Real](y, z)
    y = z
    y * y = x
}

/// The square root of a product of nonnegative reals is the product of their square-root values.
theorem sqrt_mul(a: Real, b: Real, x: Real, y: Real) {
    a >= Real.0 and b >= Real.0 and a.sqrt = Option.some(x) and b.sqrt = Option.some(y)
    implies (a * b).sqrt = Option.some(x * y)
} by {
    mul_nonneg(a, b)
    a * b >= Real.0
    sqrt_value_nonneg(a, x)
    sqrt_value_nonneg(b, y)
    x >= Real.0
    y >= Real.0
    mul_nonneg(x, y)
    x * y >= Real.0
    sqrt_value_mul_self(a, x)
    sqrt_value_mul_self(b, y)
    x * x = a
    y * y = b
    (x * y) * (x * y) = x * x * (y * y)
    x * x * (y * y) = a * b
    (x * y) * (x * y) = a * b
    sqrt_unique_nonneg(a * b, x * y)
    (a * b).sqrt = Option.some(x * y)
}

/// The square root is monotone on nonnegative reals, for returned values.
theorem sqrt_monotone(a: Real, b: Real, x: Real, y: Real) {
    Real.0 <= a and a <= b and a.sqrt = Option.some(x) and b.sqrt = Option.some(y)
    implies x <= y
} by {
    lte_trans(Real.0, a, b)
    Real.0 <= b
    b >= Real.0
    sqrt_value_nonneg(a, x)
    sqrt_value_nonneg(b, y)
    x >= Real.0
    y >= Real.0
    sqrt_value_mul_self(a, x)
    sqrt_value_mul_self(b, y)
    x * x = a
    y * y = b
    x * x <= y * y
    square_le_square_of_nonneg(x, y)
    x <= y
}

/// The geometric mean of two nonnegative reals is at most their arithmetic mean.
theorem sqrt_mul_le_arithmetic_mean(a: Real, b: Real) {
    a >= Real.0 and b >= Real.0 implies exists(g: Real) {
        (a * b).sqrt = Option.some(g) and g <= Real.one_half * (a + b)
    }
} by {
    sqrt_mul_self(a)
    sqrt_mul_self(b)
    let x: Real satisfy {
        a.sqrt = Option.some(x) and x * x = a
    }
    let y: Real satisfy {
        b.sqrt = Option.some(y) and y * y = b
    }
    sqrt_mul(a, b, x, y)
    (a * b).sqrt = Option.some(x * y)
    double_mul_le_square_sum(x, y)
    x * y + x * y <= x * x + y * y
    double_le_imp_le_half(x * y, x * x + y * y)
    x * y <= Real.one_half * (x * x + y * y)
    x * x + y * y = a + b
    Real.one_half * (x * x + y * y) = Real.one_half * (a + b)
    x * y <= Real.one_half * (a + b)
    exists(g: Real) {
        g = x * y and (a * b).sqrt = Option.some(g) and g <= Real.one_half * (a + b)
    }
}

/// The geometric mean of two nonnegative reals is at most their arithmetic mean, division form.
theorem sqrt_mul_le_arithmetic_mean_div(a: Real, b: Real) {
    a >= Real.0 and b >= Real.0 implies exists(g: Real) {
        (a * b).sqrt = Option.some(g) and g <= (a + b) / (Real.1 + Real.1)
    }
} by {
    sqrt_mul_le_arithmetic_mean(a, b)
    let g: Real satisfy {
        (a * b).sqrt = Option.some(g) and g <= Real.one_half * (a + b)
    }
    one_half_eq_one_div_two
    Real.one_half = Real.1 / (Real.1 + Real.1)
    Real.one_half * (a + b) = (Real.1 / (Real.1 + Real.1)) * (a + b)
    (Real.1 / (Real.1 + Real.1)) * (a + b) = (a + b) * (Real.1 / (Real.1 + Real.1))
    (a + b) * (Real.1 / (Real.1 + Real.1)) = (a + b) / (Real.1 + Real.1)
    g <= (a + b) / (Real.1 + Real.1)
    exists(witness: Real) {
        witness = g and (a * b).sqrt = Option.some(witness) and witness <= (a + b) / (Real.1 + Real.1)
    }
}

/// Equality in binary AM-GM for positive reals forces equality of the inputs.
theorem sqrt_mul_eq_arithmetic_mean_imp_eq(a: Real, b: Real) {
    a > Real.0 and b > Real.0 and (a * b).sqrt = Option.some((a + b) / (Real.1 + Real.1))
    implies a = b
} by {
    sqrt_mul_self(a)
    sqrt_mul_self(b)
    let x: Real satisfy {
        a.sqrt = Option.some(x) and x * x = a
    }
    let y: Real satisfy {
        b.sqrt = Option.some(y) and y * y = b
    }
    a >= Real.0
    b >= Real.0
    sqrt_mul(a, b, x, y)
    (a * b).sqrt = Option.some(x * y)
    Option.some(x * y) = Option.some((a + b) / (Real.1 + Real.1))
    some_injective[Real](x * y, (a + b) / (Real.1 + Real.1))
    one_half_eq_one_div_two
    Real.one_half = Real.1 / (Real.1 + Real.1)
    x * y = (a + b) / (Real.1 + Real.1)
    (a + b) / (Real.1 + Real.1) = Real.one_half * (a + b)
    x * y = Real.one_half * (a + b)
    a + b = x * x + y * y
    x * y = Real.one_half * (x * x + y * y)
    x * y + x * y = Real.one_half * (x * x + y * y) + Real.one_half * (x * x + y * y)
    Real.one_half * (x * x + y * y) + Real.one_half * (x * x + y * y) =
        (Real.one_half + Real.one_half) * (x * x + y * y)
    one_half_plus_one_half
    Real.one_half + Real.one_half = Real.1
    (Real.one_half + Real.one_half) * (x * x + y * y) = Real.1 * (x * x + y * y)
    Real.1 * (x * x + y * y) = x * x + y * y
    x * y + x * y = x * x + y * y
    double_mul_eq_square_sum_imp_eq(x, y)
    x = y
    x * x = y * y
    a = b
}
