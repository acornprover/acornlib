from data.basic.set import Set, intersection_contains_eq, intersection_contains_intro
from real.real_field import Real
from real.topology import interior_point_intro, is_interior_point, is_open_set,
    open_set_interior_point

/// A ball with radius bounded by `eps1` is contained in `s` when the `eps1`-ball is.
theorem smaller_ball_subset_left(s: Set[Real], x: Real, eps: Real, eps1: Real) {
    eps <= eps1 and forall(y: Real) {
        y.is_close(x, eps1) implies s.contains(y)
    } implies forall(y: Real) {
        y.is_close(x, eps) implies s.contains(y)
    }
} by {
    if eps <= eps1 and forall(y: Real) { y.is_close(x, eps1) implies s.contains(y) } {
        forall(y: Real) {
            if y.is_close(x, eps) {
                (y - x).abs < eps
                (y - x).abs < eps1
                y.is_close(x, eps1)
                s.contains(y)
            }
        }
    }
}

/// A common ball contained in two sets is contained in their intersection.
theorem ball_subset_intersection(s: Set[Real], t: Set[Real], x: Real, eps: Real) {
    forall(y: Real) {
        y.is_close(x, eps) implies s.contains(y)
    } and forall(y: Real) {
        y.is_close(x, eps) implies t.contains(y)
    } implies forall(y: Real) {
        y.is_close(x, eps) implies s.intersection(t).contains(y)
    }
} by {
    if forall(y: Real) { y.is_close(x, eps) implies s.contains(y) } and
       forall(y: Real) { y.is_close(x, eps) implies t.contains(y) } {
        forall(y: Real) {
            if y.is_close(x, eps) {
                s.contains(y)
                t.contains(y)
                intersection_contains_intro(s, t, y)
                s.intersection(t).contains(y)
            }
        }
    }
}

/// Interior points are closed under binary intersection.
theorem interior_point_intersection(s: Set[Real], t: Set[Real], x: Real) {
    is_interior_point(s, x) and is_interior_point(t, x) implies
    is_interior_point(s.intersection(t), x)
} by {
    if is_interior_point(s, x) and is_interior_point(t, x) {
        s.contains(x)
        t.contains(x)
        intersection_contains_intro(s, t, x)
        let eps1: Real satisfy {
            eps1.is_positive and forall(y: Real) {
                y.is_close(x, eps1) implies s.contains(y)
            }
        }
        let eps2: Real satisfy {
            eps2.is_positive and forall(y: Real) {
                y.is_close(x, eps2) implies t.contains(y)
            }
        }
        let eps = eps1.min(eps2)
        eps.is_positive
        eps <= eps1
        eps <= eps2
        smaller_ball_subset_left(s, x, eps, eps1)
        smaller_ball_subset_left(t, x, eps, eps2)
        ball_subset_intersection(s, t, x, eps)
        interior_point_intro(s.intersection(t), x, eps)
        is_interior_point(s.intersection(t), x)
    }
}

/// The intersection of two open sets is open.
theorem intersection_of_open_is_open(s: Set[Real], t: Set[Real]) {
    is_open_set(s) and is_open_set(t) implies is_open_set(s.intersection(t))
} by {
    if is_open_set(s) and is_open_set(t) {
        forall(x: Real) {
            if s.intersection(t).contains(x) {
                intersection_contains_eq(s, t, x)
                s.contains(x)
                t.contains(x)
                open_set_interior_point(s, x)
                open_set_interior_point(t, x)
                interior_point_intersection(s, t, x)
            }
        }
        is_open_set(s.intersection(t))
    }
}
