from data.basic.function_algebra import pointwise_mul
from data.basic.functions import compose, function_eq_transport_predicate_rev, function_extensionality
from real.derivative_basic import difference_quotient, differentiable_at,
    has_derivative_at, sub_ne_zero_of_ne
from real.derivative_chain import derivative_compose
from real.derivative_linear import derivative_pointwise_mul_const,
    differentiable_pointwise_mul_const
from real.derivative_product import derivative_pointwise_mul,
    differentiable_pointwise_mul
from real.derivative_rules import div_add_distrib, neg_div
from ordered_field import inverse_of_nonnegative_is_nonnegative
from real.real_base import Real, abs_neg, abs_not_neg, lt_trans, lte_lt_trans
from real.real_field import div_cancel_common, div_le_of_mul_le, div_mul_cancel_right,
    mul_left_cancel
from real.real_ring import exists_small_mul_variant_2, lt_mul_pos_left, lt_mul_pos_right,
    lte_mul_nonneg_right, mul_abs
from real.real_seq import close_and_lt_imp_close, eps_lt_half, eps_smaller_than_both,
    neq_imp_abs_diff_pos
from real.real_series import triangle_ineq
from algebra.field.field import inverse_dist, inverse_not_zero, mul_not_zero

/// The reciprocal real function.
define reciprocal_real(x: Real) -> Real {
    x.inverse
}

/// The pointwise quotient of two real-valued functions.
define pointwise_div_real(f: Real -> Real, g: Real -> Real, x: Real) -> Real {
    f(x) / g(x)
}

/// The pointwise reciprocal of a real-valued function.
define pointwise_reciprocal_real(g: Real -> Real, x: Real) -> Real {
    g(x).inverse
}

/// The reciprocal function agrees with the inverse operation.
theorem reciprocal_real_apply(x: Real) {
    reciprocal_real(x) = x.inverse
}

/// The pointwise quotient agrees with division of values.
theorem pointwise_div_real_apply(f: Real -> Real, g: Real -> Real, x: Real) {
    pointwise_div_real(f, g, x) = f(x) / g(x)
}

/// The pointwise reciprocal agrees with the inverse of the value.
theorem pointwise_reciprocal_real_apply(g: Real -> Real, x: Real) {
    pointwise_reciprocal_real(g, x) = g(x).inverse
}

/// The pointwise reciprocal is the composition with the reciprocal real function.
theorem pointwise_reciprocal_real_eq_compose(g: Real -> Real) {
    pointwise_reciprocal_real(g) = compose(reciprocal_real, g)
} by {
    forall(x: Real) {
        pointwise_reciprocal_real(g, x) = g(x).inverse
        reciprocal_real(g(x)) = g(x).inverse
        compose(reciprocal_real, g, x) = reciprocal_real(g(x))
        pointwise_reciprocal_real(g, x) = compose(reciprocal_real, g, x)
    }
    function_extensionality(pointwise_reciprocal_real(g), compose(reciprocal_real, g))
}

/// The pointwise quotient is multiplication by the pointwise reciprocal.
theorem pointwise_div_real_eq_mul_reciprocal(f: Real -> Real, g: Real -> Real) {
    pointwise_div_real(f, g) = pointwise_mul(f, pointwise_reciprocal_real(g))
} by {
    forall(x: Real) {
        pointwise_div_real(f, g, x) = f(x) / g(x)
        pointwise_reciprocal_real(g, x) = g(x).inverse
        pointwise_mul(f, pointwise_reciprocal_real(g), x) = f(x) * pointwise_reciprocal_real(g, x)
        f(x) / g(x) = f(x) * g(x).inverse
        pointwise_div_real(f, g, x) = pointwise_mul(f, pointwise_reciprocal_real(g), x)
    }
    function_extensionality(pointwise_div_real(f, g), pointwise_mul(f, pointwise_reciprocal_real(g)))
}

/// Division by a constant is multiplication by that constant's inverse.
theorem pointwise_div_const_eq_mul_inverse(f: Real -> Real, c: Real) {
    pointwise_div_real(f, constant[Real, Real](c)) = pointwise_mul(f, constant[Real, Real](c.inverse))
} by {
    forall(x: Real) {
        pointwise_div_real(f, constant[Real, Real](c), x) = f(x) / constant[Real, Real](c, x)
        constant[Real, Real](c, x) = c
        constant[Real, Real](c.inverse, x) = c.inverse
        pointwise_mul(f, constant[Real, Real](c.inverse), x) = f(x) * c.inverse
        f(x) / c = f(x) * c.inverse
        pointwise_div_real(f, constant[Real, Real](c), x) = pointwise_mul(f, constant[Real, Real](c.inverse), x)
    }
    function_extensionality(
        pointwise_div_real(f, constant[Real, Real](c)),
        pointwise_mul(f, constant[Real, Real](c.inverse))
    )
}

/// Reversing the order of a subtraction negates it.
theorem sub_reverse_neg(a: Real, b: Real) {
    a - b = -(b - a)
} by {
    a - b = a + -b
    b - a = b + -a
    -(b - a) = -(b + -a)
    -(b + -a) = -b + --a
    --a = a
    -b + --a = -b + a
    -b + a = a + -b
    -(b - a) = a - b
}

/// Division by two successive nonzero denominators combines the denominators.
theorem div_div_denominator(a: Real, b: Real, c: Real) {
    b != Real.0 and c != Real.0 implies (a / b) / c = a / (b * c)
} by {
    if b != Real.0 and c != Real.0 {
        mul_not_zero[Real](b, c)
        b * c != Real.0
        (a / b) / c = (a * b.inverse) * c.inverse
        (a * b.inverse) * c.inverse = a * (b.inverse * c.inverse)
        inverse_dist[Real](b, c)
        (b * c).inverse = b.inverse * c.inverse
        a * (b.inverse * c.inverse) = a * (b * c).inverse
        a / (b * c) = a * (b * c).inverse
        (a / b) / c = a / (b * c)
    }
}

/// A negated common numerator cancels against a matching denominator factor.
theorem neg_div_mul_cancel_right(a: Real, b: Real) {
    a != Real.0 and b != Real.0 implies (-a) / (a * b) = -Real.1 / b
} by {
    if a != Real.0 and b != Real.0 {
        div_mul_cancel_right(a, b)
        a / (b * a) = Real.1 / b
        b * a = a * b
        a / (a * b) = Real.1 / b
        neg_div(a, a * b)
        (-a) / (a * b) = -(a / (a * b))
        (-a) / (a * b) = -(Real.1 / b)
        neg_div(Real.1, b)
        (-Real.1) / b = -(Real.1 / b)
        -Real.1 / b = -(Real.1 / b)
        (-a) / (a * b) = -Real.1 / b
    }
}

/// Points sufficiently close to a nonzero real stay bounded away from zero in absolute value.
theorem abs_lower_bound_near_nonzero(x0: Real) {
    x0 != Real.0 implies exists(delta: Real) {
        delta.is_positive and forall(x: Real) {
            x.is_close(x0, delta) implies delta <= x.abs
        }
    }
} by {
    if x0 != Real.0 {
        neq_imp_abs_diff_pos(x0, Real.0)
        (x0 - Real.0).abs.is_positive
        x0 - Real.0 = x0
        x0.abs.is_positive
        eps_lt_half(x0.abs)
        let delta: Real satisfy {
            delta.is_positive and delta + delta < x0.abs
        }
        forall(x: Real) {
            if x.is_close(x0, delta) {
                if not delta <= x.abs {
                    x.abs < delta
                    x.is_close(x0, delta) = (x - x0).abs < delta
                    (x - x0).abs < delta
                    sub_reverse_neg(x0, x)
                    x0 - x = -(x - x0)
                    abs_neg(x - x0)
                    (x0 - x).abs = (x - x0).abs
                    (x0 - x).abs < delta
                    x0 = (x0 - x) + x
                    triangle_ineq(x0 - x, x)
                    x0.abs <= (x0 - x).abs + x.abs
                    (x0 - x).abs + x.abs < delta + delta
                    lte_lt_trans(x0.abs, (x0 - x).abs + x.abs, delta + delta)
                    x0.abs < delta + delta
                    false
                }
            }
        }
        exists(delta2: Real) {
            delta2.is_positive and forall(x: Real) {
                x.is_close(x0, delta2) implies delta2 <= x.abs
            }
        }
    }
}

/// Inverses are locally bounded in absolute value near a nonzero real.
theorem inverse_local_abs_bound(x0: Real) {
    x0 != Real.0 implies exists(delta: Real, bound: Real) {
        delta.is_positive and bound.is_positive and forall(x: Real) {
            x.is_close(x0, delta) implies x != Real.0 and x.inverse.abs <= bound
        }
    }
} by {
    if x0 != Real.0 {
        abs_lower_bound_near_nonzero(x0)
        let delta: Real satisfy {
            delta.is_positive and forall(x: Real) {
                x.is_close(x0, delta) implies delta <= x.abs
            }
        }
        let bound = Real.1 / delta
        delta != Real.0
        Real.0 <= delta
        inverse_of_nonnegative_is_nonnegative[Real](delta)
        Real.0 <= delta.inverse
        if delta.inverse = Real.0 {
            delta * delta.inverse = Real.1
            delta * Real.0 = Real.0
            Real.1 = Real.0
            false
        }
        delta.inverse.is_positive
        Real.1 / delta = delta.inverse
        bound.is_positive
        forall(x: Real) {
            if x.is_close(x0, delta) {
                delta <= x.abs
                x.abs.is_positive
                if x = Real.0 {
                    x.abs = Real.0
                    false
                }
                x != Real.0
                abs_not_neg(x.inverse)
                not x.inverse.abs.is_negative
                x.inverse.abs * delta <= x.inverse.abs * x.abs
                mul_abs(x.inverse, x)
                x.inverse.abs * x.abs = (x.inverse * x).abs
                x.inverse * x = Real.1
                Real.1.abs = Real.1
                (x.inverse * x).abs = Real.1
                x.inverse.abs * delta <= Real.1
                div_le_of_mul_le(x.inverse.abs, delta, Real.1)
                x.inverse.abs <= Real.1 / delta
                x.inverse.abs <= bound
                x != Real.0 and x.inverse.abs <= bound
            }
        }
        delta.is_positive and bound.is_positive and forall(x: Real) {
            x.is_close(x0, delta) implies x != Real.0 and x.inverse.abs <= bound
        }
        exists(delta2: Real, bound2: Real) {
            delta2.is_positive and bound2.is_positive and forall(x: Real) {
                x.is_close(x0, delta2) implies x != Real.0 and x.inverse.abs <= bound2
            }
        }
    }
}

/// The difference of two nonzero inverses is the reversed difference divided by their product.
theorem inverse_sub_inverse(a: Real, b: Real) {
    a != Real.0 and b != Real.0 implies a.inverse - b.inverse = (b - a) / (a * b)
} by {
    if a != Real.0 and b != Real.0 {
        mul_not_zero[Real](a, b)
        a * b != Real.0
        a * a.inverse = Real.1
        b * b.inverse = Real.1
        (a * b) * a.inverse = b
        (a * b) * b.inverse = a
        (a * b) * (a.inverse - b.inverse) = (a * b) * a.inverse - (a * b) * b.inverse
        (a * b) * (a.inverse - b.inverse) = b - a
        mul_left_cancel(b - a, a * b, a.inverse - b.inverse)
        (b - a) / (a * b) = a.inverse - b.inverse
    }
}

/// The difference quotient of the reciprocal function has the usual nonzero split form.
theorem difference_quotient_reciprocal_real(x0: Real, x: Real) {
    x0 != Real.0 and x != Real.0 and x != x0 implies
    difference_quotient(reciprocal_real, x0, x) = (x0 - x) / (x * x0) / (x - x0)
} by {
    if x0 != Real.0 and x != Real.0 and x != x0 {
        reciprocal_real(x) = x.inverse
        reciprocal_real(x0) = x0.inverse
        inverse_sub_inverse(x, x0)
        x.inverse - x0.inverse = (x0 - x) / (x * x0)
        reciprocal_real(x) - reciprocal_real(x0) = (x0 - x) / (x * x0)
        difference_quotient(reciprocal_real, x0, x) =
            (reciprocal_real(x) - reciprocal_real(x0)) / (x - x0)
        difference_quotient(reciprocal_real, x0, x) = (x0 - x) / (x * x0) / (x - x0)
    }
}

/// The difference quotient of the reciprocal function is `-1 / (x * x0)` away from zero.
theorem difference_quotient_reciprocal_real_simplified(x0: Real, x: Real) {
    x0 != Real.0 and x != Real.0 and x != x0 implies
    difference_quotient(reciprocal_real, x0, x) = -Real.1 / (x * x0)
} by {
    if x0 != Real.0 and x != Real.0 and x != x0 {
        difference_quotient_reciprocal_real(x0, x)
        difference_quotient(reciprocal_real, x0, x) = (x0 - x) / (x * x0) / (x - x0)
        sub_ne_zero_of_ne(x, x0)
        x - x0 != Real.0
        mul_not_zero[Real](x, x0)
        x * x0 != Real.0
        sub_reverse_neg(x0, x)
        x0 - x = -(x - x0)
        (x0 - x) / (x * x0) / (x - x0) = (-(x - x0)) / (x * x0) / (x - x0)
        div_div_denominator(-(x - x0), x * x0, x - x0)
        (-(x - x0)) / (x * x0) / (x - x0) = (-(x - x0)) / ((x * x0) * (x - x0))
        (x * x0) * (x - x0) = (x - x0) * (x * x0)
        (-(x - x0)) / ((x * x0) * (x - x0)) = (-(x - x0)) / ((x - x0) * (x * x0))
        neg_div_mul_cancel_right(x - x0, x * x0)
        (-(x - x0)) / ((x - x0) * (x * x0)) = -Real.1 / (x * x0)
        difference_quotient(reciprocal_real, x0, x) = -Real.1 / (x * x0)
    }
}

/// The reciprocal difference quotient differs from its limiting value by an explicit product.
theorem reciprocal_difference_quotient_error(x0: Real, x: Real) {
    x0 != Real.0 and x != Real.0 and x != x0 implies
    difference_quotient(reciprocal_real, x0, x) - (-Real.1 / (x0 * x0)) =
        (x - x0) * x.inverse * x0.inverse * x0.inverse
} by {
    if x0 != Real.0 and x != Real.0 and x != x0 {
        difference_quotient_reciprocal_real_simplified(x0, x)
        difference_quotient(reciprocal_real, x0, x) = -Real.1 / (x * x0)
        mul_not_zero[Real](x, x0)
        mul_not_zero[Real](x0, x0)
        x * x0 != Real.0
        x0 * x0 != Real.0
        inverse_dist[Real](x, x0)
        (x * x0).inverse = x.inverse * x0.inverse
        inverse_dist[Real](x0, x0)
        (x0 * x0).inverse = x0.inverse * x0.inverse
        -Real.1 / (x * x0) = -(x.inverse * x0.inverse)
        -Real.1 / (x0 * x0) = -(x0.inverse * x0.inverse)
        difference_quotient(reciprocal_real, x0, x) - (-Real.1 / (x0 * x0)) =
            -(x.inverse * x0.inverse) - -(x0.inverse * x0.inverse)
        -(x.inverse * x0.inverse) - -(x0.inverse * x0.inverse) =
            x0.inverse * x0.inverse - x.inverse * x0.inverse
        x0.inverse * x0.inverse - x.inverse * x0.inverse =
            (x0.inverse - x.inverse) * x0.inverse
        inverse_sub_inverse(x0, x)
        x0.inverse - x.inverse = (x - x0) / (x0 * x)
        (x0.inverse - x.inverse) * x0.inverse = ((x - x0) / (x0 * x)) * x0.inverse
        inverse_dist[Real](x0, x)
        (x0 * x).inverse = x0.inverse * x.inverse
        ((x - x0) / (x0 * x)) * x0.inverse =
            ((x - x0) * (x0.inverse * x.inverse)) * x0.inverse
        ((x - x0) * (x0.inverse * x.inverse)) * x0.inverse =
            (x - x0) * x.inverse * x0.inverse * x0.inverse
        difference_quotient(reciprocal_real, x0, x) - (-Real.1 / (x0 * x0)) =
            (x - x0) * x.inverse * x0.inverse * x0.inverse
    }
}

/// The reciprocal difference-quotient error is bounded by a local inverse bound.
theorem reciprocal_error_abs_lt_bound(x0: Real, x: Real, bound: Real, eps_x: Real) {
    x0 != Real.0 and x.inverse.abs <= bound and bound.is_positive and eps_x.is_positive
    and (x - x0).abs < eps_x implies
    ((x - x0) * x.inverse * x0.inverse * x0.inverse).abs <
        eps_x * bound * x0.inverse.abs * x0.inverse.abs
} by {
    inverse_not_zero[Real](x0)
    x0.inverse != Real.0
    neq_imp_abs_diff_pos(x0.inverse, Real.0)
    (x0.inverse - Real.0).abs.is_positive
    x0.inverse - Real.0 = x0.inverse
    x0.inverse.abs.is_positive
    abs_not_neg(x.inverse)
    abs_not_neg(x - x0)
    not x.inverse.abs.is_negative
    not (x - x0).abs.is_negative
    not (x - x0).abs.is_negative and x.inverse.abs <= bound
    lte_mul_nonneg_right(x.inverse.abs, bound, (x - x0).abs)
    x.inverse.abs * (x - x0).abs <= bound * (x - x0).abs
    bound.is_positive and (x - x0).abs < eps_x
    lt_mul_pos_left((x - x0).abs, eps_x, bound)
    bound * (x - x0).abs < bound * eps_x
    lte_lt_trans(
        x.inverse.abs * (x - x0).abs,
        bound * (x - x0).abs,
        bound * eps_x
    )
    x.inverse.abs * (x - x0).abs < bound * eps_x
    (x - x0).abs * x.inverse.abs = x.inverse.abs * (x - x0).abs
    eps_x * bound = bound * eps_x
    (x - x0).abs * x.inverse.abs < eps_x * bound
    lt_mul_pos_right(
        (x - x0).abs * x.inverse.abs,
        eps_x * bound,
        x0.inverse.abs
    )
    ((x - x0).abs * x.inverse.abs) * x0.inverse.abs < (eps_x * bound) * x0.inverse.abs
    lt_mul_pos_right(
        ((x - x0).abs * x.inverse.abs) * x0.inverse.abs,
        (eps_x * bound) * x0.inverse.abs,
        x0.inverse.abs
    )
    (((x - x0).abs * x.inverse.abs) * x0.inverse.abs) * x0.inverse.abs < ((eps_x * bound) * x0.inverse.abs) * x0.inverse.abs
    ((eps_x * bound) * x0.inverse.abs) * x0.inverse.abs =
        eps_x * bound * x0.inverse.abs * x0.inverse.abs
    mul_abs(x - x0, x.inverse)
    ((x - x0) * x.inverse).abs = (x - x0).abs * x.inverse.abs
    mul_abs((x - x0) * x.inverse, x0.inverse)
    (((x - x0) * x.inverse) * x0.inverse).abs =
        ((x - x0).abs * x.inverse.abs) * x0.inverse.abs
    mul_abs((x - x0) * x.inverse * x0.inverse, x0.inverse)
    ((x - x0) * x.inverse * x0.inverse * x0.inverse).abs =
        (((x - x0).abs * x.inverse.abs) * x0.inverse.abs) * x0.inverse.abs
    ((x - x0) * x.inverse * x0.inverse * x0.inverse).abs < eps_x * bound * x0.inverse.abs * x0.inverse.abs
}

/// The product-form quotient derivative value is the usual single-fraction value.
theorem quotient_derivative_value_formula(f0: Real, g0: Real, df: Real, dg: Real) {
    g0 != Real.0 implies
    f0 * (((-Real.1 / (g0 * g0)) * dg)) + g0.inverse * df = (df * g0 - f0 * dg) / (g0 * g0)
} by {
    if g0 != Real.0 {
        mul_not_zero[Real](g0, g0)
        g0 * g0 != Real.0
        div_add_distrib(df * g0, -(f0 * dg), g0 * g0)
        (df * g0 + -(f0 * dg)) / (g0 * g0) =
            (df * g0) / (g0 * g0) + (-(f0 * dg)) / (g0 * g0)
        df * g0 - f0 * dg = df * g0 + -(f0 * dg)
        (df * g0 - f0 * dg) / (g0 * g0) =
            (df * g0) / (g0 * g0) + (-(f0 * dg)) / (g0 * g0)
        neg_div(f0 * dg, g0 * g0)
        (-(f0 * dg)) / (g0 * g0) = -((f0 * dg) / (g0 * g0))
        div_cancel_common(df, g0, g0)
        (df * g0) / (g0 * g0) = df / g0
        df / g0 = df * g0.inverse
        df * g0.inverse = g0.inverse * df
        (df * g0 - f0 * dg) / (g0 * g0) =
            g0.inverse * df + -((f0 * dg) / (g0 * g0))
        neg_div(Real.1, g0 * g0)
        (-Real.1) / (g0 * g0) = -(Real.1 / (g0 * g0))
        -Real.1 / (g0 * g0) = -(Real.1 / (g0 * g0))
        Real.1 / (g0 * g0) = (g0 * g0).inverse
        -Real.1 / (g0 * g0) = -(g0 * g0).inverse
        (f0 * dg) / (g0 * g0) = (f0 * dg) * (g0 * g0).inverse
        -((f0 * dg) / (g0 * g0)) = -((f0 * dg) * (g0 * g0).inverse)
        -((f0 * dg) * (g0 * g0).inverse) = -(f0 * dg) * (g0 * g0).inverse
        f0 * (((-Real.1 / (g0 * g0)) * dg)) = f0 * ((-(g0 * g0).inverse) * dg)
        (-(g0 * g0).inverse) * dg = -((g0 * g0).inverse * dg)
        f0 * (-((g0 * g0).inverse * dg)) = -(f0 * ((g0 * g0).inverse * dg))
        f0 * ((g0 * g0).inverse * dg) = (f0 * dg) * (g0 * g0).inverse
        -(f0 * ((g0 * g0).inverse * dg)) = -((f0 * dg) * (g0 * g0).inverse)
        f0 * ((-(g0 * g0).inverse) * dg) = -(f0 * dg) * (g0 * g0).inverse
        f0 * (((-Real.1 / (g0 * g0)) * dg)) = -((f0 * dg) / (g0 * g0))
        f0 * (((-Real.1 / (g0 * g0)) * dg)) + g0.inverse * df =
            -((f0 * dg) / (g0 * g0)) + g0.inverse * df
        -((f0 * dg) / (g0 * g0)) + g0.inverse * df =
            g0.inverse * df + -((f0 * dg) / (g0 * g0))
        f0 * (((-Real.1 / (g0 * g0)) * dg)) + g0.inverse * df =
            (df * g0 - f0 * dg) / (g0 * g0)
    }
}

/// The reciprocal function has derivative `-1 / x0^2` at every nonzero point.
theorem derivative_reciprocal_real(x0: Real) {
    x0 != Real.0 implies has_derivative_at(reciprocal_real, x0, -Real.1 / (x0 * x0))
} by {
    if x0 != Real.0 {
        inverse_local_abs_bound(x0)
        let (inv_delta: Real, bound: Real) satisfy {
            inv_delta.is_positive and bound.is_positive and forall(x: Real) {
                x.is_close(x0, inv_delta) implies x != Real.0 and x.inverse.abs <= bound
            }
        }
        inverse_not_zero[Real](x0)
        x0.inverse != Real.0
        neq_imp_abs_diff_pos(x0.inverse, Real.0)
        (x0.inverse - Real.0).abs.is_positive
        x0.inverse - Real.0 = x0.inverse
        x0.inverse.abs.is_positive
        let factor = bound * x0.inverse.abs * x0.inverse.abs
        Real.0 < bound
        Real.0 < x0.inverse.abs
        x0.inverse.abs.is_positive and Real.0 < bound
        lt_mul_pos_right(Real.0, bound, x0.inverse.abs)
        Real.0 * x0.inverse.abs = Real.0
        Real.0 < bound * x0.inverse.abs
        (bound * x0.inverse.abs).is_positive
        lt_mul_pos_right(Real.0, bound * x0.inverse.abs, x0.inverse.abs)
        Real.0 * x0.inverse.abs = Real.0
        Real.0 < (bound * x0.inverse.abs) * x0.inverse.abs
        factor.is_positive
        forall(eps: Real) {
            if eps.is_positive {
                exists_small_mul_variant_2(factor, eps)
                let eps_x: Real satisfy {
                    eps_x.is_positive and eps_x * factor < eps
                }
                eps_smaller_than_both(inv_delta, eps_x)
                let delta: Real satisfy {
                    delta.is_positive and delta < inv_delta and delta < eps_x
                }
                forall(x: Real) {
                    if x != x0 and x.is_close(x0, delta) {
                        close_and_lt_imp_close(x, x0, delta, inv_delta)
                        close_and_lt_imp_close(x, x0, delta, eps_x)
                        x.is_close(x0, inv_delta)
                        x.is_close(x0, eps_x)
                        x != Real.0
                        x.inverse.abs <= bound
                        x.is_close(x0, eps_x) = (x - x0).abs < eps_x
                        (x - x0).abs < eps_x
                        reciprocal_difference_quotient_error(x0, x)
                        difference_quotient(reciprocal_real, x0, x) - (-Real.1 / (x0 * x0)) =
                            (x - x0) * x.inverse * x0.inverse * x0.inverse
                        reciprocal_error_abs_lt_bound(x0, x, bound, eps_x)
                        ((x - x0) * x.inverse * x0.inverse * x0.inverse).abs < eps_x * bound * x0.inverse.abs * x0.inverse.abs
                        factor = bound * x0.inverse.abs * x0.inverse.abs
                        eps_x * factor = eps_x * (bound * x0.inverse.abs * x0.inverse.abs)
                        eps_x * bound * x0.inverse.abs * x0.inverse.abs = eps_x * factor
                        ((x - x0) * x.inverse * x0.inverse * x0.inverse).abs < eps_x * factor
                        lt_trans(
                            ((x - x0) * x.inverse * x0.inverse * x0.inverse).abs,
                            eps_x * factor,
                            eps
                        )
                        ((x - x0) * x.inverse * x0.inverse * x0.inverse).abs < eps
                        (difference_quotient(reciprocal_real, x0, x) - (-Real.1 / (x0 * x0))).abs < eps
                        difference_quotient(reciprocal_real, x0, x).is_close(-Real.1 / (x0 * x0), eps)
                    }
                }
                exists(delta2: Real) {
                    delta2.is_positive and forall(x: Real) {
                        x != x0 and x.is_close(x0, delta2)
                        implies difference_quotient(reciprocal_real, x0, x).is_close(-Real.1 / (x0 * x0), eps)
                    }
                }
            }
        }
        forall(eps2: Real) {
            eps2.is_positive implies exists(delta2: Real) {
                delta2.is_positive and forall(x: Real) {
                    x != x0 and x.is_close(x0, delta2)
                    implies difference_quotient(reciprocal_real, x0, x).is_close(-Real.1 / (x0 * x0), eps2)
                }
            }
        }
        has_derivative_at(reciprocal_real, x0, -Real.1 / (x0 * x0)) = forall(eps2: Real) {
            eps2.is_positive implies exists(delta2: Real) {
                delta2.is_positive and forall(x: Real) {
                    x != x0 and x.is_close(x0, delta2)
                    implies difference_quotient(reciprocal_real, x0, x).is_close(-Real.1 / (x0 * x0), eps2)
                }
            }
        }
        if not has_derivative_at(reciprocal_real, x0, -Real.1 / (x0 * x0)) {
            not forall(eps2: Real) {
                eps2.is_positive implies exists(delta2: Real) {
                    delta2.is_positive and forall(x: Real) {
                        x != x0 and x.is_close(x0, delta2)
                        implies difference_quotient(reciprocal_real, x0, x).is_close(-Real.1 / (x0 * x0), eps2)
                    }
                }
            }
            let bad_eps: Real satisfy {
                bad_eps.is_positive and forall(delta2: Real) {
                    not (delta2.is_positive and forall(x: Real) {
                        x != x0 and x.is_close(x0, delta2)
                        implies difference_quotient(reciprocal_real, x0, x).is_close(-Real.1 / (x0 * x0), bad_eps)
                    })
                }
            }
            exists(delta2: Real) {
                delta2.is_positive and forall(x: Real) {
                    x != x0 and x.is_close(x0, delta2)
                    implies difference_quotient(reciprocal_real, x0, x).is_close(-Real.1 / (x0 * x0), bad_eps)
                }
            }
            false
        }
        has_derivative_at(reciprocal_real, x0, -Real.1 / (x0 * x0))
    }
}

/// The reciprocal of a differentiable function has the chain-rule derivative at nonzero values.
theorem derivative_pointwise_reciprocal_real(g: Real -> Real, x0: Real, dg: Real) {
    has_derivative_at(g, x0, dg) and g(x0) != Real.0 implies
    has_derivative_at(pointwise_reciprocal_real(g), x0, (-Real.1 / (g(x0) * g(x0))) * dg)
} by {
    define derivative_pred(value: Real, h: Real -> Real) -> Bool {
        has_derivative_at(h, x0, value)
    }
    if has_derivative_at(g, x0, dg) and g(x0) != Real.0 {
        derivative_reciprocal_real(g(x0))
        has_derivative_at(reciprocal_real, g(x0), -Real.1 / (g(x0) * g(x0)))
        derivative_compose(g, reciprocal_real, x0, dg, -Real.1 / (g(x0) * g(x0)))
        has_derivative_at(
            compose(reciprocal_real, g),
            x0,
            (-Real.1 / (g(x0) * g(x0))) * dg
        )
        pointwise_reciprocal_real_eq_compose(g)
        derivative_pred((-Real.1 / (g(x0) * g(x0))) * dg, compose(reciprocal_real, g))
        function_eq_transport_predicate_rev(
            derivative_pred((-Real.1 / (g(x0) * g(x0))) * dg),
            pointwise_reciprocal_real(g),
            compose(reciprocal_real, g)
        )
        has_derivative_at(pointwise_reciprocal_real(g), x0, (-Real.1 / (g(x0) * g(x0))) * dg)
    }
}

/// The reciprocal of a differentiable function is differentiable at nonzero values.
theorem differentiable_pointwise_reciprocal_real(g: Real -> Real, x0: Real) {
    differentiable_at(g, x0) and g(x0) != Real.0 implies differentiable_at(pointwise_reciprocal_real(g), x0)
} by {
    if differentiable_at(g, x0) and g(x0) != Real.0 {
        let dg: Real satisfy {
            has_derivative_at(g, x0, dg)
        }
        derivative_pointwise_reciprocal_real(g, x0, dg)
        has_derivative_at(pointwise_reciprocal_real(g), x0, (-Real.1 / (g(x0) * g(x0))) * dg)
        exists(dinv: Real) {
            has_derivative_at(pointwise_reciprocal_real(g), x0, dinv)
        }
    }
}

/// The pointwise quotient satisfies the product-form quotient rule at nonzero denominator values.
theorem derivative_pointwise_div_product_form(
    f: Real -> Real, g: Real -> Real, x0: Real, df: Real, dg: Real
) {
    has_derivative_at(f, x0, df) and has_derivative_at(g, x0, dg) and g(x0) != Real.0 implies
    has_derivative_at(
        pointwise_div_real(f, g),
        x0,
        f(x0) * (((-Real.1 / (g(x0) * g(x0))) * dg)) + pointwise_reciprocal_real(g, x0) * df
    )
} by {
    define derivative_pred(value: Real, h: Real -> Real) -> Bool {
        has_derivative_at(h, x0, value)
    }
    if has_derivative_at(f, x0, df) and has_derivative_at(g, x0, dg) and g(x0) != Real.0 {
        derivative_pointwise_reciprocal_real(g, x0, dg)
        has_derivative_at(pointwise_reciprocal_real(g), x0, (-Real.1 / (g(x0) * g(x0))) * dg)
        derivative_pointwise_mul(f, pointwise_reciprocal_real(g), x0, df, (-Real.1 / (g(x0) * g(x0))) * dg)
        has_derivative_at(
            pointwise_mul(f, pointwise_reciprocal_real(g)),
            x0,
            f(x0) * (((-Real.1 / (g(x0) * g(x0))) * dg)) + pointwise_reciprocal_real(g, x0) * df
        )
        pointwise_div_real_eq_mul_reciprocal(f, g)
        derivative_pred(
            f(x0) * (((-Real.1 / (g(x0) * g(x0))) * dg)) + pointwise_reciprocal_real(g, x0) * df,
            pointwise_mul(f, pointwise_reciprocal_real(g))
        )
        function_eq_transport_predicate_rev(
            derivative_pred(f(x0) * (((-Real.1 / (g(x0) * g(x0))) * dg)) + pointwise_reciprocal_real(g, x0) * df),
            pointwise_div_real(f, g),
            pointwise_mul(f, pointwise_reciprocal_real(g))
        )
        has_derivative_at(
            pointwise_div_real(f, g),
            x0,
            f(x0) * (((-Real.1 / (g(x0) * g(x0))) * dg)) + pointwise_reciprocal_real(g, x0) * df
        )
    }
}

/// The pointwise quotient satisfies the usual quotient rule at nonzero denominator values.
theorem derivative_pointwise_div(
    f: Real -> Real, g: Real -> Real, x0: Real, df: Real, dg: Real
) {
    has_derivative_at(f, x0, df) and has_derivative_at(g, x0, dg) and g(x0) != Real.0 implies
    has_derivative_at(pointwise_div_real(f, g), x0, (df * g(x0) - f(x0) * dg) / (g(x0) * g(x0)))
} by {
    if has_derivative_at(f, x0, df) and has_derivative_at(g, x0, dg) and g(x0) != Real.0 {
        derivative_pointwise_div_product_form(f, g, x0, df, dg)
        has_derivative_at(
            pointwise_div_real(f, g),
            x0,
            f(x0) * (((-Real.1 / (g(x0) * g(x0))) * dg)) + pointwise_reciprocal_real(g, x0) * df
        )
        pointwise_reciprocal_real(g, x0) = g(x0).inverse
        quotient_derivative_value_formula(f(x0), g(x0), df, dg)
        f(x0) * (((-Real.1 / (g(x0) * g(x0))) * dg)) + g(x0).inverse * df =
            (df * g(x0) - f(x0) * dg) / (g(x0) * g(x0))
        f(x0) * (((-Real.1 / (g(x0) * g(x0))) * dg)) + pointwise_reciprocal_real(g, x0) * df =
            (df * g(x0) - f(x0) * dg) / (g(x0) * g(x0))
        has_derivative_at(pointwise_div_real(f, g), x0, (df * g(x0) - f(x0) * dg) / (g(x0) * g(x0)))
    }
}

/// Differentiability is preserved by quotients at nonzero denominator values.
theorem differentiable_pointwise_div(f: Real -> Real, g: Real -> Real, x0: Real) {
    differentiable_at(f, x0) and differentiable_at(g, x0) and g(x0) != Real.0
    implies differentiable_at(pointwise_div_real(f, g), x0)
} by {
    define differentiable_pred(h: Real -> Real) -> Bool {
        differentiable_at(h, x0)
    }
    if differentiable_at(f, x0) and differentiable_at(g, x0) and g(x0) != Real.0 {
        differentiable_pointwise_reciprocal_real(g, x0)
        differentiable_at(pointwise_reciprocal_real(g), x0)
        differentiable_pointwise_mul(f, pointwise_reciprocal_real(g), x0)
        differentiable_at(pointwise_mul(f, pointwise_reciprocal_real(g)), x0)
        pointwise_div_real_eq_mul_reciprocal(f, g)
        differentiable_pred(pointwise_mul(f, pointwise_reciprocal_real(g)))
        function_eq_transport_predicate_rev(
            differentiable_pred,
            pointwise_div_real(f, g),
            pointwise_mul(f, pointwise_reciprocal_real(g))
        )
        differentiable_at(pointwise_div_real(f, g), x0)
    }
}

/// A quotient follows the product rule once the reciprocal denominator has a derivative.
theorem derivative_pointwise_div_from_reciprocal(
    f: Real -> Real, g: Real -> Real, x0: Real, df: Real, dinv: Real
) {
    has_derivative_at(f, x0, df) and has_derivative_at(pointwise_reciprocal_real(g), x0, dinv)
    implies has_derivative_at(
        pointwise_div_real(f, g),
        x0,
        f(x0) * dinv + pointwise_reciprocal_real(g, x0) * df
    )
} by {
    define derivative_pred(value: Real, h: Real -> Real) -> Bool {
        has_derivative_at(h, x0, value)
    }
    if has_derivative_at(f, x0, df) and has_derivative_at(pointwise_reciprocal_real(g), x0, dinv) {
        derivative_pointwise_mul(f, pointwise_reciprocal_real(g), x0, df, dinv)
        has_derivative_at(
            pointwise_mul(f, pointwise_reciprocal_real(g)),
            x0,
            f(x0) * dinv + pointwise_reciprocal_real(g, x0) * df
        )
        pointwise_div_real_eq_mul_reciprocal(f, g)
        derivative_pred(
            f(x0) * dinv + pointwise_reciprocal_real(g, x0) * df,
            pointwise_mul(f, pointwise_reciprocal_real(g))
        )
        function_eq_transport_predicate_rev(
            derivative_pred(f(x0) * dinv + pointwise_reciprocal_real(g, x0) * df),
            pointwise_div_real(f, g),
            pointwise_mul(f, pointwise_reciprocal_real(g))
        )
        has_derivative_at(
            pointwise_div_real(f, g),
            x0,
            f(x0) * dinv + pointwise_reciprocal_real(g, x0) * df
        )
    }
}

/// Differentiability of a quotient follows from differentiability of the reciprocal denominator.
theorem differentiable_pointwise_div_from_reciprocal(f: Real -> Real, g: Real -> Real, x0: Real) {
    differentiable_at(f, x0) and differentiable_at(pointwise_reciprocal_real(g), x0)
    implies differentiable_at(pointwise_div_real(f, g), x0)
} by {
    define differentiable_pred(h: Real -> Real) -> Bool {
        differentiable_at(h, x0)
    }
    if differentiable_at(f, x0) and differentiable_at(pointwise_reciprocal_real(g), x0) {
        differentiable_pointwise_mul(f, pointwise_reciprocal_real(g), x0)
        differentiable_at(pointwise_mul(f, pointwise_reciprocal_real(g)), x0)
        pointwise_div_real_eq_mul_reciprocal(f, g)
        differentiable_pred(pointwise_mul(f, pointwise_reciprocal_real(g)))
        function_eq_transport_predicate_rev(
            differentiable_pred,
            pointwise_div_real(f, g),
            pointwise_mul(f, pointwise_reciprocal_real(g))
        )
        differentiable_at(pointwise_div_real(f, g), x0)
    }
}

/// Dividing a differentiable function by a constant divides its derivative by that constant.
theorem derivative_pointwise_div_const(f: Real -> Real, c: Real, x0: Real, d: Real) {
    has_derivative_at(f, x0, d)
    implies has_derivative_at(pointwise_div_real(f, constant[Real, Real](c)), x0, d / c)
} by {
    define derivative_pred(value: Real, h: Real -> Real) -> Bool {
        has_derivative_at(h, x0, value)
    }
    derivative_pointwise_mul_const(c.inverse, f, x0, d)
    has_derivative_at(pointwise_mul(f, constant[Real, Real](c.inverse)), x0, d * c.inverse)
    d / c = d * c.inverse
    pointwise_div_const_eq_mul_inverse(f, c)
    derivative_pred(d / c, pointwise_mul(f, constant[Real, Real](c.inverse)))
    function_eq_transport_predicate_rev(
        derivative_pred(d / c),
        pointwise_div_real(f, constant[Real, Real](c)),
        pointwise_mul(f, constant[Real, Real](c.inverse))
    )
    has_derivative_at(pointwise_div_real(f, constant[Real, Real](c)), x0, d / c)
}

/// Differentiability is preserved by division by a constant.
theorem differentiable_pointwise_div_const(f: Real -> Real, c: Real, x0: Real) {
    differentiable_at(f, x0) implies differentiable_at(pointwise_div_real(f, constant[Real, Real](c)), x0)
} by {
    define differentiable_pred(h: Real -> Real) -> Bool {
        differentiable_at(h, x0)
    }
    differentiable_pointwise_mul_const(c.inverse, f, x0)
    differentiable_at(pointwise_mul(f, constant[Real, Real](c.inverse)), x0)
    pointwise_div_const_eq_mul_inverse(f, c)
    differentiable_pred(pointwise_mul(f, constant[Real, Real](c.inverse)))
    function_eq_transport_predicate_rev(
        differentiable_pred,
        pointwise_div_real(f, constant[Real, Real](c)),
        pointwise_mul(f, constant[Real, Real](c.inverse))
    )
    differentiable_at(pointwise_div_real(f, constant[Real, Real](c)), x0)
}
