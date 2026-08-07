from order_set import closed_interval_set, closed_interval_set_contains_eq,
    left_open_interval_set, left_open_interval_set_contains_eq,
    left_open_interval_set_subset_closed_interval_set, open_interval_set,
    open_interval_set_contains_eq, open_interval_set_subset_closed_interval_set,
    right_open_interval_set, right_open_interval_set_contains_eq,
    right_open_interval_set_subset_closed_interval_set
from order import lt_of_lte_of_lt
from order import closed_interval, left_open_interval, open_interval, right_open_interval
from data.basic.set import intersection_contains_eq, intersection_contains_intro, set_ext
from real.real_field import Real
from real.topology import is_bounded_real_set, is_closed_set, is_open_set,
    subset_of_bounded_real_set_is_bounded
from real.topology_closed_open import intersection_of_closed_is_closed
from real.topology_compact import closed_bounded_real_set_is_compact, is_compact_real_set
from real.topology_open_intersection import intersection_of_open_is_open
from real.topology_rays import closed_lower_ray, closed_lower_ray_contains_eq,
    closed_lower_ray_is_closed, closed_upper_ray, closed_upper_ray_contains_eq,
    closed_upper_ray_is_closed, open_lower_ray, open_lower_ray_contains_eq,
    open_lower_ray_is_open, open_upper_ray, open_upper_ray_contains_eq,
    open_upper_ray_is_open

numerals Real

/// An open interval is the intersection of its two open bounding rays.
theorem open_interval_set_eq_open_ray_intersection(lower: Real, upper: Real) {
    open_interval_set(lower, upper) = open_upper_ray(lower).intersection(open_lower_ray(upper))
} by {
    let interval = open_interval_set(lower, upper)
    let rays = open_upper_ray(lower).intersection(open_lower_ray(upper))
    forall(x: Real) {
        if interval.contains(x) {
            open_interval_set_contains_eq(lower, upper, x)
            open_interval(lower, upper, x)
            lower < x
            x < upper
            open_upper_ray_contains_eq(lower, x)
            open_lower_ray_contains_eq(upper, x)
            open_upper_ray(lower).contains(x)
            open_lower_ray(upper).contains(x)
            intersection_contains_intro(open_upper_ray(lower), open_lower_ray(upper), x)
            rays.contains(x)
        }
        if rays.contains(x) {
            intersection_contains_eq(open_upper_ray(lower), open_lower_ray(upper), x)
            open_upper_ray(lower).contains(x)
            open_lower_ray(upper).contains(x)
            open_upper_ray_contains_eq(lower, x)
            open_lower_ray_contains_eq(upper, x)
            lower < x
            x < upper
            open_interval(lower, upper, x)
            open_interval_set_contains_eq(lower, upper, x)
            interval.contains(x)
        }
        interval.contains(x) = rays.contains(x)
    }
    set_ext(interval, rays)
}

/// A closed interval is the intersection of its two closed bounding rays.
theorem closed_interval_set_eq_closed_ray_intersection(lower: Real, upper: Real) {
    closed_interval_set(lower, upper) = closed_upper_ray(lower).intersection(closed_lower_ray(upper))
} by {
    let interval = closed_interval_set(lower, upper)
    let rays = closed_upper_ray(lower).intersection(closed_lower_ray(upper))
    forall(x: Real) {
        if interval.contains(x) {
            closed_interval_set_contains_eq(lower, upper, x)
            closed_interval(lower, upper, x)
            lower <= x
            x <= upper
            closed_upper_ray_contains_eq(lower, x)
            closed_lower_ray_contains_eq(upper, x)
            closed_upper_ray(lower).contains(x)
            closed_lower_ray(upper).contains(x)
            intersection_contains_intro(closed_upper_ray(lower), closed_lower_ray(upper), x)
            rays.contains(x)
        }
        if rays.contains(x) {
            intersection_contains_eq(closed_upper_ray(lower), closed_lower_ray(upper), x)
            closed_upper_ray(lower).contains(x)
            closed_lower_ray(upper).contains(x)
            closed_upper_ray_contains_eq(lower, x)
            closed_lower_ray_contains_eq(upper, x)
            lower <= x
            x <= upper
            closed_interval(lower, upper, x)
            closed_interval_set_contains_eq(lower, upper, x)
            interval.contains(x)
        }
        interval.contains(x) = rays.contains(x)
    }
    set_ext(interval, rays)
}

/// A left-open interval is the intersection of an open upper ray and a closed lower ray.
theorem left_open_interval_set_eq_ray_intersection(lower: Real, upper: Real) {
    left_open_interval_set(lower, upper) = open_upper_ray(lower).intersection(closed_lower_ray(upper))
} by {
    let interval = left_open_interval_set(lower, upper)
    let rays = open_upper_ray(lower).intersection(closed_lower_ray(upper))
    forall(x: Real) {
        if interval.contains(x) {
            left_open_interval_set_contains_eq(lower, upper, x)
            left_open_interval(lower, upper, x)
            lower < x
            x <= upper
            open_upper_ray_contains_eq(lower, x)
            closed_lower_ray_contains_eq(upper, x)
            open_upper_ray(lower).contains(x)
            closed_lower_ray(upper).contains(x)
            intersection_contains_intro(open_upper_ray(lower), closed_lower_ray(upper), x)
            rays.contains(x)
        }
        if rays.contains(x) {
            intersection_contains_eq(open_upper_ray(lower), closed_lower_ray(upper), x)
            open_upper_ray(lower).contains(x)
            closed_lower_ray(upper).contains(x)
            open_upper_ray_contains_eq(lower, x)
            closed_lower_ray_contains_eq(upper, x)
            lower < x
            x <= upper
            left_open_interval(lower, upper, x)
            left_open_interval_set_contains_eq(lower, upper, x)
            interval.contains(x)
        }
        interval.contains(x) = rays.contains(x)
    }
    set_ext(interval, rays)
}

/// A right-open interval is the intersection of a closed upper ray and an open lower ray.
theorem right_open_interval_set_eq_ray_intersection(lower: Real, upper: Real) {
    right_open_interval_set(lower, upper) = closed_upper_ray(lower).intersection(open_lower_ray(upper))
} by {
    let interval = right_open_interval_set(lower, upper)
    let rays = closed_upper_ray(lower).intersection(open_lower_ray(upper))
    forall(x: Real) {
        if interval.contains(x) {
            right_open_interval_set_contains_eq(lower, upper, x)
            right_open_interval(lower, upper, x)
            lower <= x
            x < upper
            closed_upper_ray_contains_eq(lower, x)
            open_lower_ray_contains_eq(upper, x)
            closed_upper_ray(lower).contains(x)
            open_lower_ray(upper).contains(x)
            intersection_contains_intro(closed_upper_ray(lower), open_lower_ray(upper), x)
            rays.contains(x)
        }
        if rays.contains(x) {
            intersection_contains_eq(closed_upper_ray(lower), open_lower_ray(upper), x)
            closed_upper_ray(lower).contains(x)
            open_lower_ray(upper).contains(x)
            closed_upper_ray_contains_eq(lower, x)
            open_lower_ray_contains_eq(upper, x)
            lower <= x
            x < upper
            right_open_interval(lower, upper, x)
            right_open_interval_set_contains_eq(lower, upper, x)
            interval.contains(x)
        }
        interval.contains(x) = rays.contains(x)
    }
    set_ext(interval, rays)
}

/// Open intervals of real numbers are open sets.
theorem open_interval_set_is_open(lower: Real, upper: Real) {
    is_open_set(open_interval_set(lower, upper))
} by {
    open_upper_ray_is_open(lower)
    open_lower_ray_is_open(upper)
    intersection_of_open_is_open(open_upper_ray(lower), open_lower_ray(upper))
    is_open_set(open_upper_ray(lower).intersection(open_lower_ray(upper)))
    open_interval_set_eq_open_ray_intersection(lower, upper)
    is_open_set(open_interval_set(lower, upper))
}

/// Closed intervals of real numbers are closed sets.
theorem closed_interval_set_is_closed(lower: Real, upper: Real) {
    is_closed_set(closed_interval_set(lower, upper))
} by {
    closed_upper_ray_is_closed(lower)
    closed_lower_ray_is_closed(upper)
    intersection_of_closed_is_closed(closed_upper_ray(lower), closed_lower_ray(upper))
    is_closed_set(closed_upper_ray(lower).intersection(closed_lower_ray(upper)))
    closed_interval_set_eq_closed_ray_intersection(lower, upper)
    is_closed_set(closed_interval_set(lower, upper))
}

/// Closed intervals of real numbers are bounded sets.
theorem closed_interval_set_is_bounded(lower: Real, upper: Real) {
    is_bounded_real_set(closed_interval_set(lower, upper))
} by {
    let endpoint_bound = lower.abs.max(upper.abs)
    let bound = endpoint_bound + Real.1
    Real.1 > Real.0
    not lower.abs.is_negative
    lower.abs >= Real.0
    lower.abs <= endpoint_bound
    upper.abs <= endpoint_bound
    endpoint_bound >= Real.0
    endpoint_bound < endpoint_bound + Real.1
    endpoint_bound < bound
    lt_of_lte_of_lt[Real](Real.0, endpoint_bound, bound)
    Real.0 < bound
    endpoint_bound <= bound
    lower.abs <= bound
    upper.abs <= bound
    bound > Real.0
    forall(x: Real) {
        if closed_interval_set(lower, upper).contains(x) {
            closed_interval_set_contains_eq(lower, upper, x)
            closed_interval(lower, upper, x)
            lower <= x
            x <= upper
            -lower.abs <= lower
            -bound <= -lower.abs
            -bound <= lower
            -bound <= x
            upper <= upper.abs
            upper.abs <= bound
            upper <= bound
            x <= bound
            -bound <= x and x <= bound
        }
    }
    exists(b: Real) {
        b > Real.0 and forall(x: Real) {
            closed_interval_set(lower, upper).contains(x) implies -b <= x and x <= b
        }
    }
}

/// Open intervals of real numbers are bounded sets.
theorem open_interval_set_is_bounded(lower: Real, upper: Real) {
    is_bounded_real_set(open_interval_set(lower, upper))
} by {
    closed_interval_set_is_bounded(lower, upper)
    is_bounded_real_set(closed_interval_set(lower, upper))
    open_interval_set_subset_closed_interval_set(lower, upper)
    open_interval_set(lower, upper).subset(closed_interval_set(lower, upper))
    subset_of_bounded_real_set_is_bounded(open_interval_set(lower, upper), closed_interval_set(lower, upper))
    is_bounded_real_set(open_interval_set(lower, upper))
}

/// Left-open intervals of real numbers are bounded sets.
theorem left_open_interval_set_is_bounded(lower: Real, upper: Real) {
    is_bounded_real_set(left_open_interval_set(lower, upper))
} by {
    closed_interval_set_is_bounded(lower, upper)
    is_bounded_real_set(closed_interval_set(lower, upper))
    left_open_interval_set_subset_closed_interval_set(lower, upper)
    left_open_interval_set(lower, upper).subset(closed_interval_set(lower, upper))
    subset_of_bounded_real_set_is_bounded(left_open_interval_set(lower, upper), closed_interval_set(lower, upper))
    is_bounded_real_set(left_open_interval_set(lower, upper))
}

/// Right-open intervals of real numbers are bounded sets.
theorem right_open_interval_set_is_bounded(lower: Real, upper: Real) {
    is_bounded_real_set(right_open_interval_set(lower, upper))
} by {
    closed_interval_set_is_bounded(lower, upper)
    is_bounded_real_set(closed_interval_set(lower, upper))
    right_open_interval_set_subset_closed_interval_set(lower, upper)
    right_open_interval_set(lower, upper).subset(closed_interval_set(lower, upper))
    subset_of_bounded_real_set_is_bounded(right_open_interval_set(lower, upper), closed_interval_set(lower, upper))
    is_bounded_real_set(right_open_interval_set(lower, upper))
}

/// Closed intervals of real numbers are compact sets.
theorem closed_interval_set_is_compact(lower: Real, upper: Real) {
    is_compact_real_set(closed_interval_set(lower, upper))
} by {
    closed_interval_set_is_closed(lower, upper)
    is_closed_set(closed_interval_set(lower, upper))
    closed_interval_set_is_bounded(lower, upper)
    is_bounded_real_set(closed_interval_set(lower, upper))
    closed_bounded_real_set_is_compact(closed_interval_set(lower, upper))
    is_compact_real_set(closed_interval_set(lower, upper))
}
