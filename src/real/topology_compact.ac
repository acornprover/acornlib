from data.basic.set import Set, sets_subset_intersection
from real.real_field import Real
from real.topology import closed_set_eq_closure, closure, empty_real_set_is_bounded,
    empty_set_is_closed, is_bounded_real_set, is_closed_set, singleton_real_set_is_bounded,
    subset_of_bounded_real_set_is_bounded, union_of_bounded_real_sets_is_bounded
from real.topology_bounded_closure import closure_of_bounded_real_set_is_bounded
from real.topology_closed_open import intersection_of_closed_is_closed
from real.topology_closure import closure_is_closed
from real.topology_closure_union import union_of_closed_is_closed
from real.topology_singleton import singleton_real_set_is_closed

/// True if a set of real numbers is closed and bounded.
define is_compact_real_set(s: Set[Real]) -> Bool {
    is_closed_set(s) and is_bounded_real_set(s)
}

/// A compact real set is closed.
theorem compact_real_set_is_closed(s: Set[Real]) {
    is_compact_real_set(s) implies is_closed_set(s)
}

/// A compact real set is bounded.
theorem compact_real_set_is_bounded(s: Set[Real]) {
    is_compact_real_set(s) implies is_bounded_real_set(s)
}

/// A closed bounded real set is compact.
theorem closed_bounded_real_set_is_compact(s: Set[Real]) {
    is_closed_set(s) and is_bounded_real_set(s) implies is_compact_real_set(s)
}

/// The empty set of real numbers is compact.
theorem empty_real_set_is_compact {
    is_compact_real_set(Set[Real].empty_set)
} by {
    empty_set_is_closed
    empty_real_set_is_bounded
    is_closed_set(Set[Real].empty_set) and is_bounded_real_set(Set[Real].empty_set)
}

/// A singleton set of real numbers is compact.
theorem singleton_real_set_is_compact(a: Real) {
    is_compact_real_set(Set[Real].singleton(a))
} by {
    singleton_real_set_is_closed(a)
    singleton_real_set_is_bounded(a)
    is_closed_set(Set[Real].singleton(a)) and is_bounded_real_set(Set[Real].singleton(a))
}

/// A closed subset of a compact real set is compact.
theorem closed_subset_of_compact_real_set_is_compact(s: Set[Real], t: Set[Real]) {
    s.subset(t) and is_closed_set(s) and is_compact_real_set(t) implies is_compact_real_set(s)
} by {
    if s.subset(t) and is_closed_set(s) and is_compact_real_set(t) {
        compact_real_set_is_bounded(t)
        is_bounded_real_set(t)
        subset_of_bounded_real_set_is_bounded(s, t)
        is_bounded_real_set(s)
        is_closed_set(s) and is_bounded_real_set(s)
        is_compact_real_set(s)
    }
}

/// The intersection of two compact real sets is compact.
theorem intersection_of_compact_real_sets_is_compact(s: Set[Real], t: Set[Real]) {
    is_compact_real_set(s) and is_compact_real_set(t) implies is_compact_real_set(s.intersection(t))
} by {
    if is_compact_real_set(s) and is_compact_real_set(t) {
        compact_real_set_is_closed(s)
        compact_real_set_is_closed(t)
        is_closed_set(s)
        is_closed_set(t)
        intersection_of_closed_is_closed(s, t)
        is_closed_set(s.intersection(t))
        sets_subset_intersection[Real](s, t)
        s.intersection(t).subset(s)
        compact_real_set_is_bounded(s)
        is_bounded_real_set(s)
        subset_of_bounded_real_set_is_bounded(s.intersection(t), s)
        is_bounded_real_set(s.intersection(t))
        is_closed_set(s.intersection(t)) and is_bounded_real_set(s.intersection(t))
        is_compact_real_set(s.intersection(t))
    }
}

/// The union of two compact real sets is compact.
theorem union_of_compact_real_sets_is_compact(s: Set[Real], t: Set[Real]) {
    is_compact_real_set(s) and is_compact_real_set(t) implies is_compact_real_set(s.union(t))
} by {
    if is_compact_real_set(s) and is_compact_real_set(t) {
        compact_real_set_is_closed(s)
        compact_real_set_is_closed(t)
        is_closed_set(s)
        is_closed_set(t)
        union_of_closed_is_closed(s, t)
        is_closed_set(s.union(t))
        compact_real_set_is_bounded(s)
        compact_real_set_is_bounded(t)
        is_bounded_real_set(s)
        is_bounded_real_set(t)
        union_of_bounded_real_sets_is_bounded(s, t)
        is_bounded_real_set(s.union(t))
        is_closed_set(s.union(t)) and is_bounded_real_set(s.union(t))
        is_compact_real_set(s.union(t))
    }
}

/// A compact real set is equal to its closure.
theorem compact_real_set_eq_closure(s: Set[Real]) {
    is_compact_real_set(s) implies s = closure(s)
} by {
    if is_compact_real_set(s) {
        compact_real_set_is_closed(s)
        closed_set_eq_closure(s)
    }
}

/// The closure of a compact real set is compact.
theorem closure_of_compact_real_set_is_compact(s: Set[Real]) {
    is_compact_real_set(s) implies is_compact_real_set(closure(s))
} by {
    if is_compact_real_set(s) {
        closure_is_closed(s)
        is_closed_set(closure(s))
        compact_real_set_is_bounded(s)
        is_bounded_real_set(s)
        closure_of_bounded_real_set_is_bounded(s)
        is_bounded_real_set(closure(s))
        is_closed_set(closure(s)) and is_bounded_real_set(closure(s))
        is_compact_real_set(closure(s))
    }
}
