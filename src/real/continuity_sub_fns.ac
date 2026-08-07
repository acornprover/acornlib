from real.continuity_algebra import add_fns_continuous_at, add_fns_continuous
from real.continuity_pointwise import continuous_at_pointwise_neg,
    continuous_pointwise_neg
from real.continuity_base import Real, add_fns, continuous, continuous_at
from data.basic.function_algebra import pointwise_neg

/// Pointwise subtraction of two real-valued functions.
define sub_fns(f: Real -> Real, g: Real -> Real, x: Real) -> Real {
    f(x) - g(x)
}

/// Pointwise subtraction agrees with addition of the pointwise negation.
theorem sub_fns_eq_add_fns_neg(f: Real -> Real, g: Real -> Real) {
    sub_fns(f, g) = add_fns[Real](f, pointwise_neg[Real, Real](g))
} by {
    forall(x: Real) {
        pointwise_neg[Real, Real](g, x) = -g(x)
        add_fns[Real](f, pointwise_neg[Real, Real](g), x) = f(x) + (-g(x))
        f(x) + (-g(x)) = f(x) - g(x)
        sub_fns(f, g, x) = f(x) - g(x)
        sub_fns(f, g, x) = add_fns[Real](f, pointwise_neg[Real, Real](g), x)
    }
}

/// The pointwise difference of two functions continuous at x is continuous at x.
theorem sub_fns_continuous_at(f: Real -> Real, g: Real -> Real, x: Real) {
    continuous_at(f, x) and continuous_at(g, x)
    implies continuous_at(sub_fns(f, g), x)
} by {
    if continuous_at(f, x) and continuous_at(g, x) {
        continuous_at_pointwise_neg(g, x)
        continuous_at(pointwise_neg[Real, Real](g), x)
        add_fns_continuous_at(f, pointwise_neg[Real, Real](g), x)
        continuous_at(add_fns[Real](f, pointwise_neg[Real, Real](g)), x)
        sub_fns_eq_add_fns_neg(f, g)
        continuous_at(sub_fns(f, g), x)
    }
}

/// The pointwise difference of two continuous real functions is continuous.
theorem sub_fns_continuous(f: Real -> Real, g: Real -> Real) {
    continuous(f) and continuous(g)
    implies continuous(sub_fns(f, g))
} by {
    if continuous(f) and continuous(g) {
        continuous_pointwise_neg(g)
        continuous(pointwise_neg[Real, Real](g))
        add_fns_continuous(f, pointwise_neg[Real, Real](g))
        continuous(add_fns[Real](f, pointwise_neg[Real, Real](g)))
        sub_fns_eq_add_fns_neg(f, g)
        continuous(sub_fns(f, g))
    }
}
