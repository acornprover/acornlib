/// The root test for real series.
///
/// If the sequence of n-th roots of the absolute values,
///
///     root_seq(a, n) = |a(n)|^(1/(n+1)),
///
/// converges to l < 1, then Σ a(n) converges absolutely.  (The exponent
/// 1/(n+1) avoids division by zero at n = 0; the test is unaffected since
/// convergence depends only on the tail.)
///
/// The proof route is the classical one:
///
///   (1) choose alpha with l < alpha < 1 and 0 <= alpha; the root sequence is
///       eventually strictly below alpha,
///   (2) the root-power identity (x^(1/p))^p = x (real.rpow machinery from
///       holder_minkowski) gives |a(n)| = root_seq(a, n)^(n+1), so the bound
///       root_seq(a, n) < alpha yields |a(n)| <= alpha^(n+1),
///   (3) the geometric majorant alpha^k dominates |a(k)| eventually, so
///       eventually_abs_le_geometric_absolutely_converges applies.

from nat import Nat, from_nat, pow_zero, lte_cancel_suc, alt_suc_ne_zero, only_zero_lte_zero, lte_suc_suc
from real.real_field import Real
from real.real_base import abs_gte_zero, lte_self, gt_zero_imp_pos, pos_gt_zero, lte_lt_trans
from real.real_ring import converges, limit, converges_to, real_mul_comm
from real.real_series import pow_nonneg
from real.abs_conv import absolutely_converges, abs_fn
from real.series_tests import converges_to_strict_ub, exists_alpha_between
from real.series_geometric_majorant import geometric_majorant, eventually_abs_le_geometric_absolutely_converges
from real.holder_minkowski import abs_pow_value, abs_pow_value_root, abs_pow_value_nonneg
from real.log import rpow_nat
from real.exp_inequalities import one_div_one_div
from real.exp import pow_suc, zero_pow_pos, exp_pos
from real.harmonic import from_nat_suc_pos_real, real_one_div_pos
from real.finite_product_mean import from_nat_real_ne_zero_of_ne_zero
from order import lt_imp_lte, lte_antisymm, not_gt_imp_lte, lte_trans
from ordered_field import mul_le_mul_of_nonneg_right, mul_le_mul_of_nonneg_left

numerals Real
numerals Nat

/// The (n+1)-th root of |a(n)|, i.e. |a(n)|^(1/(n+1)).
define root_seq(a: Nat -> Real, n: Nat) -> Real {
    abs_pow_value(a, Real.1 / from_nat[Real](n.suc), n.suc)
}

/// The exponent 1/(n+1) of the root sequence is positive.
theorem root_exp_pos(n: Nat) {
    Real.1 / from_nat[Real](n.suc) > Real.0
} by {
    from_nat_suc_pos_real(n)
    from_nat[Real](n.suc) > Real.0
    real_one_div_pos(from_nat[Real](n.suc))
    (Real.1 / from_nat[Real](n.suc)).is_positive
    pos_gt_zero(Real.1 / from_nat[Real](n.suc))
    Real.1 / from_nat[Real](n.suc) > Real.0
}

/// The reciprocal of 1/(n+1) is n+1.
theorem root_exp_recip(n: Nat) {
    Real.1 / (Real.1 / from_nat[Real](n.suc)) = from_nat[Real](n.suc)
} by {
    alt_suc_ne_zero(n)
    n.suc != Nat.0
    from_nat_real_ne_zero_of_ne_zero(n.suc)
    from_nat[Real](n.suc) != Real.0
    one_div_one_div(from_nat[Real](n.suc))
    Real.1 / (Real.1 / from_nat[Real](n.suc)) = from_nat[Real](n.suc)
}

/// The (n+1)-th power of the root is the absolute value: (|a|^(1/(n+1)))^(n+1) = |a|.
theorem root_pow_eq_abs(a: Nat -> Real, n: Nat) {
    root_seq(a, n).pow(n.suc) = a(n.suc).abs
} by {
    if a(n.suc).abs > Real.0 {
        // The root is positive, so the root-power identity applies.
        abs_pow_value(a, Real.1 / from_nat[Real](n.suc), n.suc) =
            ((Real.1 / from_nat[Real](n.suc)) * (a(n.suc).abs).log.get_or_else(Real.0)).exp
        exp_pos((Real.1 / from_nat[Real](n.suc)) * (a(n.suc).abs).log.get_or_else(Real.0))
        ((Real.1 / from_nat[Real](n.suc)) * (a(n.suc).abs).log.get_or_else(Real.0)).exp > Real.0
        abs_pow_value(a, Real.1 / from_nat[Real](n.suc), n.suc) > Real.0
        root_seq(a, n) = abs_pow_value(a, Real.1 / from_nat[Real](n.suc), n.suc)
        root_seq(a, n) > Real.0
        rpow_nat(root_seq(a, n), n.suc)
        (root_seq(a, n)).rpow(from_nat[Real](n.suc)) =
            Option.some(root_seq(a, n).pow(n.suc))
        root_exp_pos(n)
        abs_pow_value_root(a, Real.1 / from_nat[Real](n.suc), n.suc)
        (abs_pow_value(a, Real.1 / from_nat[Real](n.suc), n.suc)).rpow(Real.1 / (Real.1 / from_nat[Real](n.suc))) = Option.some(a(n.suc).abs)
        root_exp_recip(n)
        Real.1 / (Real.1 / from_nat[Real](n.suc)) = from_nat[Real](n.suc)
        (root_seq(a, n)).rpow(from_nat[Real](n.suc)) = Option.some(a(n.suc).abs)
        some_injective[Real](root_seq(a, n).pow(n.suc), a(n.suc).abs)
        root_seq(a, n).pow(n.suc) = a(n.suc).abs
    } else {
        // |a(n+1)| = 0, so the root is zero.
        not a(n.suc).abs > Real.0
        not_gt_imp_lte(a(n.suc).abs, Real.0)
        a(n.suc).abs <= Real.0
        abs_gte_zero(a(n.suc))
        Real.0 <= a(n.suc).abs
        lte_antisymm(a(n.suc).abs, Real.0)
        a(n.suc).abs = Real.0
        abs_pow_value(a, Real.1 / from_nat[Real](n.suc), n.suc) = Real.0
        root_seq(a, n) = abs_pow_value(a, Real.1 / from_nat[Real](n.suc), n.suc)
        root_seq(a, n) = Real.0
        root_seq(a, n).pow(n.suc) = Real.0.pow(n.suc)
        zero_pow_pos(n.suc)
        Nat.0 <= n
        lte_suc_suc(Nat.0, n)
        Nat.0.suc <= n.suc
        Nat.0.suc = Nat.1
        n.suc >= Nat.1
        Real.0.pow(n.suc) = Real.0
        root_seq(a, n).pow(n.suc) = Real.0
        a(n.suc).abs = Real.0
        root_seq(a, n).pow(n.suc) = a(n.suc).abs
    }
}

/// Powers are monotone in the base for nonnegative bases:
/// 0 <= x <= y implies x^n <= y^n.
theorem pow_lte_base(x: Real, y: Real, n: Nat) {
    Real.0 <= x and x <= y implies x.pow(n) <= y.pow(n)
} by {
    define p(k: Nat) -> Bool {
        Real.0 <= x and x <= y implies x.pow(k) <= y.pow(k)
    }
    pow_zero(x)
    pow_zero(y)
    x.pow(Nat.0) = Real.1
    y.pow(Nat.0) = Real.1
    lte_self(Real.1)
    Real.1 <= Real.1
    p(Nat.0)
    forall(k: Nat) {
        if p(k) {
            if Real.0 <= x and x <= y {
                x.pow(k) <= y.pow(k)
                pow_nonneg(x, k)
                Real.0 <= x.pow(k)
                lte_trans(Real.0, x, y)
                Real.0 <= y
                pow_nonneg(y, k)
                Real.0 <= y.pow(k)
                pow_suc(x, k)
                pow_suc(y, k)
                x.pow(k.suc) = x * x.pow(k)
                y.pow(k.suc) = y * y.pow(k)
                real_mul_comm(x, x.pow(k))
                x * x.pow(k) = x.pow(k) * x
                x.pow(k.suc) = x.pow(k) * x
                real_mul_comm(y, y.pow(k))
                y * y.pow(k) = y.pow(k) * y
                y.pow(k.suc) = y.pow(k) * y
                mul_le_mul_of_nonneg_right(x.pow(k), y.pow(k), x)
                x.pow(k) <= y.pow(k) and Real.0 <= x implies x.pow(k) * x <= y.pow(k) * x
                x.pow(k) * x <= y.pow(k) * x
                x.pow(k.suc) <= y.pow(k) * x
                mul_le_mul_of_nonneg_left(x, y, y.pow(k))
                x <= y and Real.0 <= y.pow(k) implies y.pow(k) * x <= y.pow(k) * y
                y.pow(k) * x <= y.pow(k) * y
                lte_trans(x.pow(k.suc), y.pow(k) * x, y.pow(k) * y)
                x.pow(k.suc) <= y.pow(k) * y
                y.pow(k) * y = y.pow(k.suc)
                x.pow(k.suc) <= y.pow(k.suc)
            }
            p(k.suc)
        }
    }
    p(Nat.0) and forall(k: Nat) { p(k) implies p(k.suc) }
    Nat.induction(p)
    p(n)
}

/// A root strictly below alpha bounds the absolute value by alpha^(n+1).
theorem root_lt_alpha_imp_abs_le(a: Nat -> Real, n: Nat, alpha: Real) {
    root_seq(a, n) < alpha and Real.0 <= alpha
    implies a(n.suc).abs <= alpha.pow(n.suc)
} by {
    if root_seq(a, n) < alpha and Real.0 <= alpha {
        root_exp_pos(n)
        abs_pow_value_nonneg(a, Real.1 / from_nat[Real](n.suc), n.suc)
        Real.1 / from_nat[Real](n.suc) > Real.0 implies abs_pow_value(a, Real.1 / from_nat[Real](n.suc), n.suc) >= Real.0
        abs_pow_value(a, Real.1 / from_nat[Real](n.suc), n.suc) >= Real.0
        root_seq(a, n) = abs_pow_value(a, Real.1 / from_nat[Real](n.suc), n.suc)
        root_seq(a, n) >= Real.0
        lt_imp_lte(root_seq(a, n), alpha)
        root_seq(a, n) <= alpha
        pow_lte_base(root_seq(a, n), alpha, n.suc)
        root_seq(a, n).pow(n.suc) <= alpha.pow(n.suc)
        root_pow_eq_abs(a, n)
        root_seq(a, n).pow(n.suc) = a(n.suc).abs
        a(n.suc).abs <= alpha.pow(n.suc)
    }
}

/// A root bound at every index from n0 on dominates the tail |a(k)| <= alpha^k
/// from n0 + 1 on by the geometric majorant.
theorem root_test_eventual_majorant(a: Nat -> Real, alpha: Real, n0: Nat) {
    (forall(n: Nat) { n0 <= n implies root_seq(a, n) < alpha })
    and Real.0 <= alpha
    implies
    forall(k: Nat) { n0.suc <= k implies abs_fn(a)(k) <= geometric_majorant(Real.1, alpha, k) }
} by {
    if forall(n: Nat) { n0 <= n implies root_seq(a, n) < alpha } and Real.0 <= alpha {
        // The bound at every index from n0 + 1 on, by induction.
        define p(k: Nat) -> Bool {
            n0.suc <= k implies abs_fn(a)(k) <= geometric_majorant(Real.1, alpha, k)
        }
        // Base case: n0 + 1 <= 0 is impossible.
        if n0.suc <= Nat.0 {
            only_zero_lte_zero(n0.suc)
            n0.suc = Nat.0
            alt_suc_ne_zero(n0)
            n0.suc != Nat.0
            false
        }
        p(Nat.0)
        // Inductive step: the root bound at index k dominates |a(k+1)|.
        forall(k: Nat) {
            if p(k) {
                if n0.suc <= k.suc {
                    lte_cancel_suc(n0, k)
                    n0 <= k
                    forall(n: Nat) { n0 <= n implies root_seq(a, n) < alpha }
                    root_seq(a, k) < alpha
                    root_lt_alpha_imp_abs_le(a, k, alpha)
                    a(k.suc).abs <= alpha.pow(k.suc)
                    abs_fn(a)(k.suc) = a(k.suc).abs
                    abs_fn(a)(k.suc) <= alpha.pow(k.suc)
                    geometric_majorant(Real.1, alpha, k.suc) = Real.1 * alpha.pow(k.suc)
                    Real.1 * alpha.pow(k.suc) = alpha.pow(k.suc)
                    abs_fn(a)(k.suc) <= geometric_majorant(Real.1, alpha, k.suc)
                }
                p(k.suc)
            }
        }
        p(Nat.0) and forall(k: Nat) { p(k) implies p(k.suc) }
        Nat.induction(p)
        forall(k: Nat) {
            p(k)
            p(k) = (n0.suc <= k implies abs_fn(a)(k) <= geometric_majorant(Real.1, alpha, k))
            n0.suc <= k implies abs_fn(a)(k) <= geometric_majorant(Real.1, alpha, k)
        }
    }
}

/// The root test: if |a(n)|^(1/(n+1)) converges to l < 1, then Σ a(n)
/// converges absolutely.
theorem root_test(a: Nat -> Real, l: Real) {
    converges_to(root_seq(a), l) and l < Real.1
    implies absolutely_converges(a)
} by {
    if converges_to(root_seq(a), l) and l < Real.1 {
        // Choose alpha with l < alpha < 1 and 0 <= alpha.
        exists_alpha_between(l)
        let alpha: Real satisfy {
            l < alpha and alpha < Real.1 and Real.0 <= alpha
        }
        // The root sequence is eventually strictly below alpha.
        converges_to_strict_ub(root_seq(a), l, alpha)
        exists(n: Nat) {
            forall(i: Nat) {
                n <= i implies root_seq(a)(i) < alpha
            }
        }
        let big_n: Nat satisfy {
            forall(i: Nat) {
                big_n <= i implies root_seq(a)(i) < alpha
            }
        }
        (forall(i: Nat) { big_n <= i implies root_seq(a, i) < alpha }) and Real.0 <= alpha
        root_test_eventual_majorant(a, alpha, big_n)
        forall(k: Nat) {
            big_n.suc <= k implies abs_fn(a)(k) <= geometric_majorant(Real.1, alpha, k)
        }
        // The geometric majorant converges since 0 <= alpha < 1.
        Real.1.is_positive
        pos_gt_zero(Real.1)
        Real.1 > Real.0
        Real.0 < Real.1
        lt_imp_lte(Real.0, Real.1)
        Real.0 <= Real.1
        eventually_abs_le_geometric_absolutely_converges(a, Real.1, alpha, big_n.suc)
        absolutely_converges(a)
    }
}
