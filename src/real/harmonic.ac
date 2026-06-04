from nat import Nat
from ordered_field import inverse_on_positive_flips_inequality
from rat import Rat
from real.exp import suc_pos
from real.real_base import Real
from real.real_ring import from_nat_is_from_rat
from nat import from_nat
from int import Int

/// The denominator of a harmonic-series term is positive.
theorem from_nat_suc_pos_real(n: Nat) {
    from_nat[Real](n.suc) > Real.0
} by {
    from_nat_is_from_rat(n.suc)
    suc_pos(n)
    from_nat[Real](n.suc) = Real.from_rat(Rat.from_nat(n.suc))
    Real.from_rat(Rat.from_nat(n.suc)) > Real.0
}

/// The kth harmonic-series term, indexed from zero.
define harmonic(k: Nat) -> Real {
    Real.1 / from_nat[Real](k.suc)
}

/// The reciprocal of a positive real is positive.
theorem real_one_div_pos(a: Real) {
    a > Real.0 implies Real.1 / a > Real.0
} by {
    a.is_positive
    a.inverse.is_positive
    Real.1.is_positive
    (Real.1 * a.inverse).is_positive
    Real.1 / a = Real.1 * a.inverse
    (Real.1 / a).is_positive
    Real.1 / a > Real.0
}

/// Harmonic-series terms are positive.
theorem harmonic_pos(k: Nat) {
    harmonic(k) > Real.0
} by {
    from_nat[Real](k.suc) > Real.0
    Real.1 / from_nat[Real](k.suc) > Real.0
    harmonic(k) = Real.1 / from_nat[Real](k.suc)
    harmonic(k) > Real.0
}

/// Harmonic-series terms are nonnegative.
theorem harmonic_nonnegative(k: Nat) {
    harmonic(k) >= Real.0
} by {
    harmonic_pos(k)
}

/// Taking inverses reverses a strict inequality between positive reals.
theorem real_inverse_antitone_pos_strict(a: Real, b: Real) {
    a < b and a > Real.0 implies b.inverse < a.inverse
} by {
    Real.0 < a
    Real.0 < b
    inverse_on_positive_flips_inequality[Real](a, b)
}

/// Taking reciprocals reverses a non-strict inequality between positive reals.
theorem real_recip_antitone_pos(a: Real, b: Real) {
    a > Real.0 and a <= b implies Real.1 / b <= Real.1 / a
} by {
    a < b or a = b
    if a < b {
        real_inverse_antitone_pos_strict(a, b)
        b.inverse < a.inverse
        Real.1 / b = b.inverse
        Real.1 / a = a.inverse
        Real.1 / b <= Real.1 / a
    }
    if a = b {
        Real.1 / b = Real.1 / a
        Real.1 / b <= Real.1 / a
    }
}

/// Conversion from natural numbers to rationals preserves non-strict order.
theorem rat_from_nat_lte_of_nat_lte(m: Nat, n: Nat) {
    m <= n implies Rat.from_nat(m) <= Rat.from_nat(n)
} by {
    Int.from_nat(m) <= Int.from_nat(n)
    Rat.from_int(Int.from_nat(m)) <= Rat.from_int(Int.from_nat(n))
    Rat.from_nat(m) = Rat.from_int(Int.from_nat(m))
    Rat.from_nat(n) = Rat.from_int(Int.from_nat(n))
}

/// A small explicit bridge for the generic `from_nat` instance on `Real`.
theorem touch_real_from_nat_instance(k: Nat) {
    from_nat[Real](k) = Real.from_rat(Rat.from_nat(k))
} by {
    from_nat_is_from_rat(k)
}

/// Conversion from natural numbers to reals preserves successor inequalities.
theorem from_nat_real_suc_lte_of_nat_lte(m: Nat, n: Nat) {
    m <= n implies from_nat[Real](m.suc) <= from_nat[Real](n.suc)
} by {
    m.suc <= n.suc
    Rat.from_nat(m.suc) <= Rat.from_nat(n.suc)
    Real.from_rat(Rat.from_nat(m.suc)) <= Real.from_rat(Rat.from_nat(n.suc))
    from_nat[Real](m.suc) = Real.from_rat(Rat.from_nat(m.suc))
    from_nat[Real](n.suc) = Real.from_rat(Rat.from_nat(n.suc))
}

/// Harmonic denominators turn natural-number inequalities into reciprocal bounds.
theorem nat_recip_antitone_suc(m: Nat, n: Nat) {
    m <= n implies Real.1 / from_nat[Real](n.suc) <= Real.1 / from_nat[Real](m.suc)
} by {
    from_nat[Real](m.suc) > Real.0
    from_nat[Real](m.suc) <= from_nat[Real](n.suc)
    Real.1 / from_nat[Real](n.suc) <= Real.1 / from_nat[Real](m.suc)
}

/// A denominator comparison witness for terms in the dyadic block `[n, 2n)`.
theorem nat_block_den_eq(n: Nat, k: Nat) {
    n > Nat.0 and k < n implies exists(q: Nat) { n + n = (n + k + q).suc }
} by {
    let r: Nat satisfy { k + r = n and r != Nat.0 }
    let q: Nat satisfy { r = q.suc }
    k + q.suc = n
    k + q + Nat.1 = n
    n + n = n + (k + q.suc)
    n + (k + q.suc) = n + (k + q).suc
    n + (k + q).suc = (n + (k + q)).suc
    n + (k + q) = n + k + q
    n + n = (n + k + q).suc
    exists(q0: Nat) { q0 = q and n + n = (n + k + q0).suc }
}

/// Each harmonic term in the dyadic block `[n, 2n)` is bounded below by `1 / (2n)`.
theorem harmonic_block_term_lower(n: Nat, k: Nat) {
    n > Nat.0 and k < n implies harmonic(n + k) >= Real.1 / from_nat[Real](n + n)
} by {
    let q: Nat satisfy { n + n = (n + k + q).suc }
    n + k <= n + k + q
    Real.1 / from_nat[Real]((n + k + q).suc) <= Real.1 / from_nat[Real]((n + k).suc)
    from_nat[Real](n + n) = from_nat[Real]((n + k + q).suc)
    Real.1 / from_nat[Real](n + n) = Real.1 / from_nat[Real]((n + k + q).suc)
    harmonic(n + k) = Real.1 / from_nat[Real]((n + k).suc)
    Real.1 / from_nat[Real](n + n) <= harmonic(n + k)
}
