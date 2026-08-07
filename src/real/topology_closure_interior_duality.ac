from data.basic.set import Set, compl_contains_eq, compl_of_compl_is_self, double_inclusion
from real.real_field import Real
from real.topology import adherent_point_eps, closure, closure_contains_eq,
    interior, interior_contains_eq, interior_point_intro, is_adherent_point_of_set,
    is_eps_adherent_to_set, is_interior_point, point_in_closure_if_in_set
from real.topology_complements import missing_eps_adherent_excludes_member,
    not_adherent_has_missing_eps

/// A point outside the closure of a set lies in the interior of the complement.
theorem closure_complement_member_interior_complement(s: Set[Real], x: Real) {
    closure(s).c.contains(x) implies interior(s.c).contains(x)
} by {
    if closure(s).c.contains(x) {
        compl_contains_eq(closure(s), x)
        not closure(s).contains(x)
        closure_contains_eq(s, x)
        not is_adherent_point_of_set(s, x)
        not_adherent_has_missing_eps(s, x)
        let eps: Real satisfy {
            eps.is_positive and not is_eps_adherent_to_set(s, x, eps)
        }
        if s.contains(x) {
            point_in_closure_if_in_set(s, x)
            false
        }
        not s.contains(x)
        compl_contains_eq(s, x)
        s.c.contains(x)
        forall(y: Real) {
            if y.is_close(x, eps) {
                missing_eps_adherent_excludes_member(s, x, eps, y)
                not s.contains(y)
                compl_contains_eq(s, y)
                s.c.contains(y)
            }
        }
        interior_point_intro(s.c, x, eps)
        is_interior_point(s.c, x)
        interior_contains_eq(s.c, x)
        interior(s.c).contains(x)
    }
}

/// A point in the interior of a complement lies outside the closure.
theorem interior_complement_member_closure_complement(s: Set[Real], x: Real) {
    interior(s.c).contains(x) implies closure(s).c.contains(x)
} by {
    if interior(s.c).contains(x) {
        interior_contains_eq(s.c, x)
        is_interior_point(s.c, x)
        let eps: Real satisfy {
            eps.is_positive and forall(y: Real) {
                y.is_close(x, eps) implies s.c.contains(y)
            }
        }
        if closure(s).contains(x) {
            closure_contains_eq(s, x)
            is_adherent_point_of_set(s, x)
            adherent_point_eps(s, x, eps)
            is_eps_adherent_to_set(s, x, eps)
            let y: Real satisfy {
                s.contains(y) and y.is_close(x, eps)
            }
            s.c.contains(y)
            compl_contains_eq(s, y)
            not s.contains(y)
            false
        }
        not closure(s).contains(x)
        compl_contains_eq(closure(s), x)
        closure(s).c.contains(x)
    }
}

/// The complement of a closure is contained in the interior of the complement.
theorem closure_complement_subset_interior_complement(s: Set[Real]) {
    closure(s).c.subset(interior(s.c))
} by {
    forall(x: Real) {
        if closure(s).c.contains(x) {
            closure_complement_member_interior_complement(s, x)
        }
    }
}

/// The interior of a complement is contained in the complement of the closure.
theorem interior_complement_subset_closure_complement(s: Set[Real]) {
    interior(s.c).subset(closure(s).c)
} by {
    forall(x: Real) {
        if interior(s.c).contains(x) {
            interior_complement_member_closure_complement(s, x)
        }
    }
}

/// Interior of a complement is the complement of the closure.
theorem interior_complement_eq_closure_complement(s: Set[Real]) {
    interior(s.c) = closure(s).c
} by {
    interior_complement_subset_closure_complement(s)
    closure_complement_subset_interior_complement(s)
    double_inclusion(interior(s.c), closure(s).c)
}

/// Interior is the complement of the closure of the complement.
theorem interior_eq_closure_complement_complement(s: Set[Real]) {
    interior(s) = closure(s.c).c
} by {
    compl_of_compl_is_self[Real](s)
    s.c.c = s
    interior_complement_eq_closure_complement(s.c)
    interior(s.c.c) = closure(s.c).c
    interior(s) = closure(s.c).c
}

/// Closure of a complement is the complement of the interior.
theorem closure_complement_eq_interior_complement(s: Set[Real]) {
    closure(s.c) = interior(s).c
} by {
    interior_eq_closure_complement_complement(s)
    interior(s) = closure(s.c).c
    interior(s).c = closure(s.c).c.c
    compl_of_compl_is_self[Real](closure(s.c))
    closure(s.c).c.c = closure(s.c)
    interior(s).c = closure(s.c)
    closure(s.c) = interior(s).c
}
