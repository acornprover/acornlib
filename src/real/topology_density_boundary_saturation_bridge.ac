from data.basic.set import Set, intersection_comm, intersection_with_universal_is_self
from real.real_field import Real
from real.topology import closure, is_closed_set, is_open_set
from real.topology_boundary import boundary
from real.topology_closed_open import complement_of_open_is_closed
from real.topology_codense import is_codense_real_set
from real.topology_dense import is_dense_real_set
from real.topology_interior_closure_idempotent import closed_set_iff_eq_closure

/// A dense real set has boundary equal to the closure of its complement.
theorem boundary_of_dense_real_set_eq_closure_complement(s: Set[Real]) {
    is_dense_real_set(s) implies boundary(s) = closure(s.c)
} by {
    if is_dense_real_set(s) {
        is_dense_real_set(s) = (closure(s) = Set[Real].universal_set)
        closure(s) = Set[Real].universal_set
        boundary(s) = closure(s).intersection(closure(s.c))
        boundary(s) = Set[Real].universal_set.intersection(closure(s.c))
        intersection_comm[Real](Set[Real].universal_set, closure(s.c))
        Set[Real].universal_set.intersection(closure(s.c)) = closure(s.c).intersection(Set[Real].universal_set)
        intersection_with_universal_is_self[Real](closure(s.c))
        closure(s.c).intersection(Set[Real].universal_set) = closure(s.c)
        boundary(s) = closure(s.c)
    }
}

/// A codense real set has boundary equal to its closure.
theorem boundary_of_codense_real_set_eq_closure(s: Set[Real]) {
    is_codense_real_set(s) implies boundary(s) = closure(s)
} by {
    if is_codense_real_set(s) {
        is_codense_real_set(s) = is_dense_real_set(s.c)
        is_dense_real_set(s.c)
        is_dense_real_set(s.c) = (closure(s.c) = Set[Real].universal_set)
        closure(s.c) = Set[Real].universal_set
        boundary(s) = closure(s).intersection(closure(s.c))
        boundary(s) = closure(s).intersection(Set[Real].universal_set)
        intersection_with_universal_is_self[Real](closure(s))
        closure(s).intersection(Set[Real].universal_set) = closure(s)
        boundary(s) = closure(s)
    }
}

/// A dense open real set has boundary equal to its complement.
theorem boundary_of_dense_open_real_set_eq_complement(s: Set[Real]) {
    is_dense_real_set(s) and is_open_set(s) implies boundary(s) = s.c
} by {
    if is_dense_real_set(s) and is_open_set(s) {
        boundary_of_dense_real_set_eq_closure_complement(s)
        boundary(s) = closure(s.c)
        complement_of_open_is_closed(s)
        is_closed_set(s.c)
        closed_set_iff_eq_closure(s.c)
        s.c = closure(s.c)
        boundary(s) = s.c
    }
}

/// A closed codense real set is equal to its boundary.
theorem boundary_of_closed_codense_real_set_eq_self(s: Set[Real]) {
    is_closed_set(s) and is_codense_real_set(s) implies boundary(s) = s
} by {
    if is_closed_set(s) and is_codense_real_set(s) {
        boundary_of_codense_real_set_eq_closure(s)
        boundary(s) = closure(s)
        closed_set_iff_eq_closure(s)
        s = closure(s)
        boundary(s) = s
    }
}
