from data.basic.set import Set, double_inclusion
from real.real_field import Real
from real.real_seq import close_and_lt_imp_close, eps_lt_half, is_close_triangle
from real.topology import adherent_point_eps, adherent_point_intro, closure,
    closure_contains_imp_adherent, closure_contains_of_adherent,
    closure_mono, closure_subset_of_closed_set, eps_adherent_of_contains_close,
    is_adherent_point_of_set, is_closed_set, is_eps_adherent_to_set

/// An adherent point of a closure is epsilon-adherent to the original set.
theorem adherent_closure_eps_adherent(s: Set[Real], x: Real, eps: Real) {
    is_adherent_point_of_set(closure(s), x) and eps.is_positive
    implies is_eps_adherent_to_set(s, x, eps)
} by {
    if is_adherent_point_of_set(closure(s), x) and eps.is_positive {
        eps_lt_half(eps)
        let eps2: Real satisfy {
            eps2.is_positive and eps2 + eps2 < eps
        }
        adherent_point_eps(closure(s), x, eps2)
        is_eps_adherent_to_set(closure(s), x, eps2)
        let y: Real satisfy {
            closure(s).contains(y) and y.is_close(x, eps2)
        }
        closure_contains_imp_adherent(s, y)
        is_adherent_point_of_set(s, y)
        adherent_point_eps(s, y, eps2)
        is_eps_adherent_to_set(s, y, eps2)
        let z: Real satisfy {
            s.contains(z) and z.is_close(y, eps2)
        }
        is_close_triangle(z, y, x, eps2, eps2)
        z.is_close(x, eps2 + eps2)
        close_and_lt_imp_close(z, x, eps2 + eps2, eps)
        z.is_close(x, eps)
        eps_adherent_of_contains_close(s, x, z, eps)
        is_eps_adherent_to_set(s, x, eps)
    }
}

/// An adherent point of the closure is adherent to the original set.
theorem adherent_closure_imp_adherent(s: Set[Real], x: Real) {
    is_adherent_point_of_set(closure(s), x) implies is_adherent_point_of_set(s, x)
} by {
    if is_adherent_point_of_set(closure(s), x) {
        forall(eps: Real) {
            if eps.is_positive {
                adherent_closure_eps_adherent(s, x, eps)
            }
        }
        adherent_point_intro(s, x)
        is_adherent_point_of_set(s, x)
    }
}

/// Every adherent point of a closure lies in that closure.
theorem closure_contains_adherent_point(s: Set[Real], x: Real) {
    is_adherent_point_of_set(closure(s), x) implies closure(s).contains(x)
} by {
    if is_adherent_point_of_set(closure(s), x) {
        adherent_closure_imp_adherent(s, x)
        closure_contains_of_adherent(s, x)
    }
}

/// The closure of a set is closed.
theorem closure_is_closed(s: Set[Real]) {
    is_closed_set(closure(s))
} by {
    forall(x: Real) {
        if is_adherent_point_of_set(closure(s), x) {
            closure_contains_adherent_point(s, x)
        }
    }
}

/// Closure is idempotent.
theorem closure_idempotent(s: Set[Real]) {
    closure(closure(s)) = closure(s)
} by {
    closure_is_closed(s)
    is_closed_set(closure(s))
    closure_subset_of_closed_set(closure(s))
    closure(closure(s)).subset(closure(s))
    closure_mono(s, closure(s))
    closure(s).subset(closure(closure(s)))
    double_inclusion(closure(closure(s)), closure(s))
}
