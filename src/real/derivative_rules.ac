from data.basic.function_algebra import pointwise_add, pointwise_neg
from real.derivative_basic import difference_quotient, has_derivative_at,
    differentiable_at, sub_ne_zero_of_ne
from real.real_base import Real
from real.real_seq import add_close, close_and_lt_imp_close, eps_lt_half,
    eps_smaller_than_both, neg_is_close

/// Division distributes over addition when the denominator is nonzero.
theorem div_add_distrib(a: Real, b: Real, c: Real) {
    c != Real.0 implies (a + b) / c = a / c + b / c
} by {
    if c != Real.0 {
        (a + b) * c.inverse = a * c.inverse + b * c.inverse
    }
}

/// Division commutes with negation in the numerator.
theorem neg_div(a: Real, b: Real) {
    (-a) / b = -(a / b)
} by {
    (-a) * b.inverse = -(a * b.inverse)
}

/// Subtracting two sums is the sum of the corresponding differences.
theorem sub_add_distrib(a: Real, b: Real, c: Real, d: Real) {
    (a + b) - (c + d) = (a - c) + (b - d)
} by {
    -(c + d) = -c + -d
    (a + b) - (c + d) = a + b + -c + -d
    a - c = a + -c
    b - d = b + -d
    (a - c) + (b - d) = a + -c + b + -d
    -c + b = b + -c
    a + -c + b = a + b + -c
    a + -c + b + -d = a + b + -c + -d
}

/// Negating both terms in a difference negates the difference.
theorem neg_sub_neg(a: Real, b: Real) {
    -a - -b = -(a - b)
} by {
    --b = b
    -a - -b = -a + --b
    -a - -b = -a + b
    a - b = a + -b
    -(a + -b) = -a + --b
    -(a - b) = -a + b
}

/// The difference quotient of a pointwise negation is the negated difference quotient.
theorem difference_quotient_pointwise_neg(f: Real -> Real, x0: Real, x: Real) {
    x != x0 implies
    difference_quotient(pointwise_neg(f), x0, x) = -difference_quotient(f, x0, x)
} by {
    if x != x0 {
        pointwise_neg(f, x) = -f(x)
        pointwise_neg(f, x0) = -f(x0)
        pointwise_neg(f, x) - pointwise_neg(f, x0) = -f(x) - -f(x0)
        neg_sub_neg(f(x), f(x0))
        -f(x) - -f(x0) = -(f(x) - f(x0))
        difference_quotient(pointwise_neg(f), x0, x) = (-(f(x) - f(x0))) / (x - x0)
        neg_div(f(x) - f(x0), x - x0)
        (-(f(x) - f(x0))) / (x - x0) = -((f(x) - f(x0)) / (x - x0))
        difference_quotient(f, x0, x) = (f(x) - f(x0)) / (x - x0)
        difference_quotient(pointwise_neg(f), x0, x) = -difference_quotient(f, x0, x)
    }
}

/// The difference quotient of a pointwise sum is the sum of the difference quotients.
theorem difference_quotient_pointwise_add(f: Real -> Real, g: Real -> Real, x0: Real, x: Real) {
    x != x0 implies
    difference_quotient(pointwise_add(f, g), x0, x) =
        difference_quotient(f, x0, x) + difference_quotient(g, x0, x)
} by {
    if x != x0 {
        sub_ne_zero_of_ne(x, x0)
        x - x0 != Real.0
        pointwise_add(f, g, x) = f(x) + g(x)
        pointwise_add(f, g, x0) = f(x0) + g(x0)
        let a = f(x) - f(x0)
        let b = g(x) - g(x0)
        sub_add_distrib(f(x), g(x), f(x0), g(x0))
        (f(x) + g(x)) - (f(x0) + g(x0)) = a + b
        difference_quotient(pointwise_add(f, g), x0, x) = (a + b) / (x - x0)
        div_add_distrib(a, b, x - x0)
        (a + b) / (x - x0) = a / (x - x0) + b / (x - x0)
        difference_quotient(f, x0, x) = a / (x - x0)
        difference_quotient(g, x0, x) = b / (x - x0)
        difference_quotient(pointwise_add(f, g), x0, x) =
            difference_quotient(f, x0, x) + difference_quotient(g, x0, x)
    }
}

/// Pointwise negation negates derivatives.
theorem derivative_pointwise_neg(f: Real -> Real, x0: Real, d: Real) {
    has_derivative_at(f, x0, d) implies has_derivative_at(pointwise_neg(f), x0, -d)
} by {
    has_derivative_at(f, x0, d) = forall(eps: Real) {
        eps.is_positive implies exists(delta: Real) {
            delta.is_positive and forall(x: Real) {
                x != x0 and x.is_close(x0, delta)
                implies difference_quotient(f, x0, x).is_close(d, eps)
            }
        }
    }
    forall(eps: Real) {
        if eps.is_positive {
            let delta: Real satisfy {
                delta.is_positive and forall(x: Real) {
                    x != x0 and x.is_close(x0, delta)
                    implies difference_quotient(f, x0, x).is_close(d, eps)
                }
            }
            forall(x: Real) {
                if x != x0 and x.is_close(x0, delta) {
                    difference_quotient(f, x0, x).is_close(d, eps)
                    neg_is_close(difference_quotient(f, x0, x), d, eps)
                    (-difference_quotient(f, x0, x)).is_close(-d, eps)
                    difference_quotient_pointwise_neg(f, x0, x)
                    difference_quotient(pointwise_neg(f), x0, x) = -difference_quotient(f, x0, x)
                    difference_quotient(pointwise_neg(f), x0, x).is_close(-d, eps)
                }
            }
            exists(delta2: Real) {
                delta2.is_positive and forall(x: Real) {
                    x != x0 and x.is_close(x0, delta2)
                    implies difference_quotient(pointwise_neg(f), x0, x).is_close(-d, eps)
                }
            }
        }
    }
    has_derivative_at(pointwise_neg(f), x0, -d) = forall(eps: Real) {
        eps.is_positive implies exists(delta: Real) {
            delta.is_positive and forall(x: Real) {
                x != x0 and x.is_close(x0, delta)
                implies difference_quotient(pointwise_neg(f), x0, x).is_close(-d, eps)
            }
        }
    }
    if not has_derivative_at(pointwise_neg(f), x0, -d) {
        not forall(eps: Real) {
            eps.is_positive implies exists(delta: Real) {
                delta.is_positive and forall(x: Real) {
                    x != x0 and x.is_close(x0, delta)
                    implies difference_quotient(pointwise_neg(f), x0, x).is_close(-d, eps)
                }
            }
        }
        let bad_eps: Real satisfy {
            bad_eps.is_positive and forall(delta: Real) {
                not (delta.is_positive and forall(x: Real) {
                    x != x0 and x.is_close(x0, delta)
                    implies difference_quotient(pointwise_neg(f), x0, x).is_close(-d, bad_eps)
                })
            }
        }
        exists(delta: Real) {
            delta.is_positive and forall(x: Real) {
                x != x0 and x.is_close(x0, delta)
                implies difference_quotient(pointwise_neg(f), x0, x).is_close(-d, bad_eps)
            }
        }
        false
    }
}

/// Pointwise addition adds derivatives.
theorem derivative_pointwise_add(f: Real -> Real, g: Real -> Real, x0: Real, df: Real, dg: Real) {
    has_derivative_at(f, x0, df) and has_derivative_at(g, x0, dg)
    implies has_derivative_at(pointwise_add(f, g), x0, df + dg)
} by {
    has_derivative_at(f, x0, df) = forall(eps: Real) {
        eps.is_positive implies exists(delta: Real) {
            delta.is_positive and forall(x: Real) {
                x != x0 and x.is_close(x0, delta)
                implies difference_quotient(f, x0, x).is_close(df, eps)
            }
        }
    }
    has_derivative_at(g, x0, dg) = forall(eps: Real) {
        eps.is_positive implies exists(delta: Real) {
            delta.is_positive and forall(x: Real) {
                x != x0 and x.is_close(x0, delta)
                implies difference_quotient(g, x0, x).is_close(dg, eps)
            }
        }
    }
    forall(eps: Real) {
        if eps.is_positive {
            eps_lt_half(eps)
            let eps2: Real satisfy {
                eps2.is_positive and eps2 + eps2 < eps
            }
            let delta_f: Real satisfy {
                delta_f.is_positive and forall(x: Real) {
                    x != x0 and x.is_close(x0, delta_f)
                    implies difference_quotient(f, x0, x).is_close(df, eps2)
                }
            }
            let delta_g: Real satisfy {
                delta_g.is_positive and forall(x: Real) {
                    x != x0 and x.is_close(x0, delta_g)
                    implies difference_quotient(g, x0, x).is_close(dg, eps2)
                }
            }
            eps_smaller_than_both(delta_f, delta_g)
            let delta: Real satisfy {
                delta.is_positive and delta < delta_f and delta < delta_g
            }
            forall(x: Real) {
                if x != x0 and x.is_close(x0, delta) {
                    close_and_lt_imp_close(x, x0, delta, delta_f)
                    close_and_lt_imp_close(x, x0, delta, delta_g)
                    x.is_close(x0, delta_f)
                    x.is_close(x0, delta_g)
                    difference_quotient(f, x0, x).is_close(df, eps2)
                    difference_quotient(g, x0, x).is_close(dg, eps2)
                    add_close(difference_quotient(f, x0, x), difference_quotient(g, x0, x),
                        df, dg, eps2, eps2)
                    (difference_quotient(f, x0, x) + difference_quotient(g, x0, x)).is_close(
                        df + dg, eps2 + eps2)
                    close_and_lt_imp_close(
                        difference_quotient(f, x0, x) + difference_quotient(g, x0, x),
                        df + dg, eps2 + eps2, eps)
                    difference_quotient_pointwise_add(f, g, x0, x)
                    difference_quotient(pointwise_add(f, g), x0, x) =
                        difference_quotient(f, x0, x) + difference_quotient(g, x0, x)
                    difference_quotient(pointwise_add(f, g), x0, x).is_close(df + dg, eps)
                }
            }
            exists(delta2: Real) {
                delta2.is_positive and forall(x: Real) {
                    x != x0 and x.is_close(x0, delta2)
                    implies difference_quotient(pointwise_add(f, g), x0, x).is_close(df + dg, eps)
                }
            }
        }
    }
    has_derivative_at(pointwise_add(f, g), x0, df + dg) = forall(eps: Real) {
        eps.is_positive implies exists(delta: Real) {
            delta.is_positive and forall(x: Real) {
                x != x0 and x.is_close(x0, delta)
                implies difference_quotient(pointwise_add(f, g), x0, x).is_close(df + dg, eps)
            }
        }
    }
    if not has_derivative_at(pointwise_add(f, g), x0, df + dg) {
        not forall(eps: Real) {
            eps.is_positive implies exists(delta: Real) {
                delta.is_positive and forall(x: Real) {
                    x != x0 and x.is_close(x0, delta)
                    implies difference_quotient(pointwise_add(f, g), x0, x).is_close(df + dg, eps)
                }
            }
        }
        let bad_eps: Real satisfy {
            bad_eps.is_positive and forall(delta: Real) {
                not (delta.is_positive and forall(x: Real) {
                    x != x0 and x.is_close(x0, delta)
                    implies difference_quotient(pointwise_add(f, g), x0, x).is_close(df + dg, bad_eps)
                })
            }
        }
        exists(delta: Real) {
            delta.is_positive and forall(x: Real) {
                x != x0 and x.is_close(x0, delta)
                implies difference_quotient(pointwise_add(f, g), x0, x).is_close(df + dg, bad_eps)
            }
        }
        false
    }
}

/// Differentiability is preserved by pointwise negation.
theorem differentiable_pointwise_neg(f: Real -> Real, x0: Real) {
    differentiable_at(f, x0) implies differentiable_at(pointwise_neg(f), x0)
} by {
    let d: Real satisfy {
        has_derivative_at(f, x0, d)
    }
    derivative_pointwise_neg(f, x0, d)
    has_derivative_at(pointwise_neg(f), x0, -d)
    exists(e: Real) {
        has_derivative_at(pointwise_neg(f), x0, e)
    }
}

/// Differentiability is preserved by pointwise addition.
theorem differentiable_pointwise_add(f: Real -> Real, g: Real -> Real, x0: Real) {
    differentiable_at(f, x0) and differentiable_at(g, x0)
    implies differentiable_at(pointwise_add(f, g), x0)
} by {
    let df: Real satisfy {
        has_derivative_at(f, x0, df)
    }
    let dg: Real satisfy {
        has_derivative_at(g, x0, dg)
    }
    derivative_pointwise_add(f, g, x0, df, dg)
    has_derivative_at(pointwise_add(f, g), x0, df + dg)
    exists(d: Real) {
        has_derivative_at(pointwise_add(f, g), x0, d)
    }
}

/// Pointwise subtraction subtracts derivatives.
theorem derivative_pointwise_sub(f: Real -> Real, g: Real -> Real, x0: Real, df: Real, dg: Real) {
    has_derivative_at(f, x0, df) and has_derivative_at(g, x0, dg)
    implies has_derivative_at(pointwise_add(f, pointwise_neg(g)), x0, df + -dg)
} by {
    derivative_pointwise_neg(g, x0, dg)
    has_derivative_at(pointwise_neg(g), x0, -dg)
    derivative_pointwise_add(f, pointwise_neg(g), x0, df, -dg)
    has_derivative_at(pointwise_add(f, pointwise_neg(g)), x0, df + -dg)
}

/// Differentiability is preserved by pointwise subtraction.
theorem differentiable_pointwise_sub(f: Real -> Real, g: Real -> Real, x0: Real) {
    differentiable_at(f, x0) and differentiable_at(g, x0)
    implies differentiable_at(pointwise_add(f, pointwise_neg(g)), x0)
} by {
    differentiable_pointwise_neg(g, x0)
    differentiable_at(pointwise_neg(g), x0)
    differentiable_pointwise_add(f, pointwise_neg(g), x0)
    differentiable_at(pointwise_add(f, pointwise_neg(g)), x0)
}
