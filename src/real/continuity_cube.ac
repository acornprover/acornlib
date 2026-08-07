from data.basic.function_algebra import pointwise_mul
from data.basic.functions import identity_fn
from real.continuity_pointwise_mul import continuous_at_pointwise_mul,
    continuous_pointwise_mul
from real.continuity_composition import continuous_imp_continuous_at
from real.continuity_sequences import identity_function_is_continuous
from real.continuity_square import continuous_at_square_real,
    continuous_square_real, square_real
from real.continuity_base import Real, continuous, continuous_at

/// The pointwise cube of a real number.
define cube_real(x: Real) -> Real {
    x * x * x
}

/// The cube function agrees with the pointwise product of the square with the identity.
theorem cube_real_eq_pointwise_mul_square_identity {
    cube_real = pointwise_mul[Real, Real](square_real, identity_fn[Real])
} by {
    forall(x: Real) {
        identity_fn[Real](x) = x
        square_real(x) = x * x
        pointwise_mul[Real, Real](square_real, identity_fn[Real], x) = square_real(x) * identity_fn[Real](x)
        pointwise_mul[Real, Real](square_real, identity_fn[Real], x) = (x * x) * x
        cube_real(x) = x * x * x
        cube_real(x) = pointwise_mul[Real, Real](square_real, identity_fn[Real], x)
    }
}

/// The cube function on the reals is continuous at each point.
theorem continuous_at_cube_real(x: Real) {
    continuous_at(cube_real, x)
} by {
    continuous_at_square_real(x)
    continuous_at(square_real, x)
    identity_function_is_continuous
    continuous_imp_continuous_at(identity_fn[Real], x)
    continuous_at(identity_fn[Real], x)
    continuous_at_pointwise_mul(square_real, identity_fn[Real], x)
    continuous_at(pointwise_mul[Real, Real](square_real, identity_fn[Real]), x)
    cube_real_eq_pointwise_mul_square_identity
    continuous_at(cube_real, x)
}

/// The cube function on the reals is continuous.
theorem continuous_cube_real {
    continuous(cube_real)
} by {
    continuous_square_real
    continuous(square_real)
    identity_function_is_continuous
    continuous(identity_fn[Real])
    continuous_pointwise_mul(square_real, identity_fn[Real])
    continuous(pointwise_mul[Real, Real](square_real, identity_fn[Real]))
    cube_real_eq_pointwise_mul_square_identity
    continuous(cube_real)
}
