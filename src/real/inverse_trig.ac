/// Inverse trigonometric functions: arctangent, arcsine and arccosine.
///
/// The tangent is a strictly increasing bijection from (-pi/2, pi/2) onto the
/// reals, and the sine is a strictly increasing bijection from [-pi/2, pi/2]
/// onto [-1, 1].  This file proves those facts and defines the inverse
/// functions `arctan`, `arcsin` and `arccos` on the principal branches, using
/// the same default-valued witness pattern as the logarithm in `log.ac`.
///
/// The main results are:
/// - `tan_strictly_increasing`, `sin_strictly_increasing`: strict monotonicity
///   on the principal branches (via the difference formulas, no derivatives).
/// - `exists_tan_of`: every real is a tangent value on (-pi/2, pi/2), proved
///   with the completeness/supremum method used for `log` (continuity of the
///   tangent plus unboundedness toward the endpoints).
/// - `tan_arctan`, `arctan_tan`, `arctan_neg`, `arctan_zero`, `arctan_one`.
/// - `arctan_continuous` and `arctan_has_derivative_at`: the arctangent has
///   derivative 1/(1 + x^2), and `arctan_add` is the addition formula
///   tan(arctan x + arctan y) = (x + y)/(1 - xy).
/// - `sin_arcsin`, `arcsin_sin`, `arcsin_neg`, `arcsin_zero`, `arcsin_one`,
///   `arcsin_neg_one`.
/// - `arcsin_continuous_at`, `arcsin_has_derivative_at` (derivative
///   1/(arcsin x).cos, with `arcsin_derivative_sqrt_form` linking it to
///   (1 - x^2).sqrt) and `arccos_has_derivative_at`.
/// - `cos_arccos`, `arccos_cos`, `arccos_add_arcsin`, and the bridge
///   identities `(1 - x^2).sqrt = (arcsin(x)).cos` (equivalently
///   `(arccos(x)).sin`) on [-1, 1].

from nat import Nat, pow_one
from rat import Rat
from order import lte_antisymm, lte_trans, lte_refl, lt_trans, lt_imp_lte, not_lt_imp_gte, not_lte_imp_gt, lt_of_lte_of_lt, lt_of_lt_of_lte, lt_imp_ne, lt_imp_ne_symm
from order_set import closed_interval_set, closed_interval_set_contains_eq, closed_interval_set_lower_le, closed_interval_set_le_upper
from order import closed_interval
from data.basic.set import Set
from data.basic.functions import function_extensionality, function_eq_transport_predicate_rev
from data.basic.function_algebra import pointwise_neg, pointwise_add
from data.basic.witness import choose_or_default, choose_or_default_spec, choose_or_default_unique_eq, exists_unique, exists_unique_intro
from algebra.field.field import mul_not_zero
from real.real_field import Real, mul_inverse, mul_div_cancel, div_mul_cancel_left, inverse_div, prod_eq_to_div_eq, zero_is_different_than_one, zero_inverse, div_cancel_common
from real.real_ring import mul_zero_left, mul_zero_right, lte_mul_nonneg_right, mul_neg_right, square_nonneg
from real.real_series import pow_nonneg
from real.real_base import abs_gte_zero, pos_imp_eq_abs, lt_add_right, lte_add_right, add_comm, neg_pos_is_neg, close_imp_bounds, neg_lt_swap_neg, lt_add_converse, neg_neg, pos_gt_zero, neg_distrib, gt_zero_imp_pos, lt_neg_swap_neg, add_lte_add, neg_lt_zero, bounds_imp_close, neg_zero, lt_add_pos, add_lt_lt, neg_lt_neg_swap_neg
from ordered_field import inverse_of_positive_is_positive, multiply_inequality_with_nonnegative_element, mul_lt_mul_of_pos_right, mul_pos_pos
from real.exp import two, two_positive, div_lt_div_pos, pow_suc
from real.harmonic import real_inverse_antitone_pos_strict
from real.trig import sin_zero, cos_zero, sin_neg, cos_neg, neg_pow_even
from real.trig_identities import sin_add, cos_add, sin_sq_add_cos_sq, div_add_same_denom, div_sub_same_denom
from real.trig_identities_deep import tan, tan_zero, tan_neg, tan_mul_cos, tan_add_basic
from real.trig_period import sin_pi_over_two_sub, cos_pi_over_two_sub, sin_add_pi_over_two, cos_add_pi_over_two
from real.pi import pi, pi_over_two, pi_pos, pi_over_two_pos, pi_over_two_lt_two, pi_lt_four, pi_eq_double_pi_over_two, cos_pi_over_two_zero, sin_pi_over_two_one, sin_pi_zero, cos_pi_neg_one, cos_nonneg_on_zero_to_pi_over_two, cos_zero_set, cos_zero_contains, cos_zero_set_contains_eq, pi_over_two_is_lower_bound, pi_over_two_le_two, sin_continuous, cos_continuous
from real.derivative_trig import sin_has_derivative_at, cos_has_derivative_at, sq_lte_base, sin_is_derivative_fn, cos_is_derivative_fn
from real.calculus_api import is_derivative_fn
from real.derivative_basic import has_derivative_at, differentiable_at, difference_quotient, sub_ne_zero_of_ne, constant_has_derivative_at
from real.derivative_quotient import pointwise_div_real, pointwise_reciprocal_real, derivative_pointwise_div, pointwise_div_real_apply, pointwise_div_real_eq_mul_reciprocal, reciprocal_real, reciprocal_real_apply
from real.derivative_rules import neg_div, derivative_pointwise_sub
from real.derivative_continuity import derivative_continuous_at, div_mul_cancel_denominator
from real.continuity_base import continuous, continuous_at, continuous_condition
from real.continuity_composition import continuous_imp_continuous_at
from real.continuity_reciprocal_div import continuous_at_pointwise_div_real, continuous_at_reciprocal_real
from real.continuity_order import continuous_at_lt_target_neighborhood, continuous_at_target_lt_neighborhood, lt_target_neighborhood_apply, target_lt_neighborhood_apply
from real.supremum import completeness, has_upper_bound, is_nonempty, is_set_supremum, is_set_upper_bound, is_set_lower_bound, set_member_le_supremum, set_supremum_le_upper_bound, set_bound_below_supremum_not_upper, set_not_upper_bound_witness
from real.real_seq import eps_smaller_than_both, lt_imp_minus_pos, sub_zero_imp_eq
from real.intermediate_value import intermediate_value_closed_interval
from real.sqrt import sqrt_unique_nonneg

numerals Real

/// One quarter of pi, the arctangent of one.
let pi_over_four = pi_over_two / two

/// Half of pi is the sum of two quarters of pi.
theorem pi_over_two_eq_two_mul_pi_over_four {
    pi_over_four + pi_over_four = pi_over_two
} by {
    pi_over_four = pi_over_two / two
    pi_over_four + pi_over_four = pi_over_two / two + pi_over_two / two
    two_positive
    two != Real.0
    div_add_same_denom(Real.1, pi_over_two, pi_over_two, two)
    Real.1 * pi_over_two / two + Real.1 * pi_over_two / two =
        Real.1 * (pi_over_two + pi_over_two) / two
    Real.1 * pi_over_two / two + Real.1 * pi_over_two / two =
        (pi_over_two + pi_over_two) / two
    pi_over_two + pi_over_two = two * pi_over_two
    pi_over_four + pi_over_four = two * pi_over_two / two
    div_mul_cancel_left(two, pi_over_two)
    two * pi_over_two / two = pi_over_two
    pi_over_four + pi_over_four = pi_over_two
}

/// A quarter of pi lies strictly between zero and half of pi.
theorem pi_over_four_pos {
    Real.0 < pi_over_four and pi_over_four < pi_over_two
} by {
    pi_over_two_pos
    Real.0 < pi_over_two
    two_positive
    two > Real.0
    div_lt_div_pos(Real.0, pi_over_two, two)
    Real.0 / two < pi_over_two / two
    Real.0 / two = Real.0
    Real.0 < pi_over_two / two
    pi_over_four = pi_over_two / two
    Real.0 < pi_over_four
    Real.1 < two
    Real.1 > Real.0
    real_inverse_antitone_pos_strict(Real.1, two)
    two.inverse < Real.1.inverse
    Real.1.inverse = Real.1
    Real.1 / two = two.inverse
    Real.1 / two < Real.1
    pi_over_two > Real.0
    mul_lt_mul_of_pos_right(Real.1 / two, Real.1, pi_over_two)
    (Real.1 / two) * pi_over_two < Real.1 * pi_over_two
    Real.1 * pi_over_two = pi_over_two
    pi_over_two * (Real.1 / two) < pi_over_two
    pi_over_two / two = pi_over_two * (Real.1 / two)
    pi_over_two / two < pi_over_two
    pi_over_four < pi_over_two
    Real.0 < pi_over_four and pi_over_four < pi_over_two
}

// Positivity of sine and cosine on the principal intervals.
//
// Cosine is nonnegative on [0, pi/2] by `cos_nonneg_on_zero_to_pi_over_two`,
// and it has no zero strictly between zero and pi/2 because pi/2 is the
// infimum of the positive zero set of cosine (`pi_over_two_is_lower_bound`).

/// Cosine has no zero in the open interval (0, pi/2).
theorem cos_ne_zero_on_pos_lt_pi_over_two(x: Real) {
    Real.0 < x and x < pi_over_two implies x.cos != Real.0
} by {
    if Real.0 < x and x < pi_over_two {
        if x.cos = Real.0 {
            lt_imp_lte(x, pi_over_two)
            x <= pi_over_two
            pi_over_two_le_two
            pi_over_two <= two
            lte_trans(x, pi_over_two, two)
            x <= two
            cos_zero_contains(x) = (Real.0 < x and x <= two and x.cos = Real.0)
            Real.0 < x and x <= two and x.cos = Real.0
            cos_zero_set_contains_eq(x)
            cos_zero_set.contains(x) = cos_zero_contains(x)
            cos_zero_set.contains(x)
            pi_over_two_is_lower_bound
            is_set_lower_bound(cos_zero_set, pi_over_two)
            is_set_lower_bound(cos_zero_set, pi_over_two) = forall(y: Real) {
                cos_zero_set.contains(y) implies pi_over_two <= y
            }
            pi_over_two <= x
            x < pi_over_two
            false
        }
        x.cos != Real.0
    }
}

/// Cosine is positive on the open interval (0, pi/2).
theorem cos_pos_on_pos_lt_pi_over_two(x: Real) {
    Real.0 < x and x < pi_over_two implies x.cos > Real.0
} by {
    if Real.0 < x and x < pi_over_two {
        lt_imp_lte(Real.0, x)
        square_nonneg(x)
        x * x >= Real.0
        Real.0 <= x * x
        lt_imp_lte(x, pi_over_two)
        x <= pi_over_two
        cos_nonneg_on_zero_to_pi_over_two(x)
        Real.0 <= x.cos
        cos_ne_zero_on_pos_lt_pi_over_two(x)
        x.cos != Real.0
        if not Real.0 < x.cos {
            not_lt_imp_gte(Real.0, x.cos)
            Real.0 >= x.cos
            x.cos <= Real.0
            lte_antisymm(Real.0, x.cos)
            Real.0 = x.cos
            x.cos = Real.0
            false
        }
        Real.0 < x.cos
        x.cos > Real.0
    }
}

/// Cosine is positive on the open interval (-pi/2, pi/2).
theorem cos_pos_on_open_interval(x: Real) {
    -pi_over_two < x and x < pi_over_two implies x.cos > Real.0
} by {
    if -pi_over_two < x and x < pi_over_two {
        if Real.0 < x {
            cos_pos_on_pos_lt_pi_over_two(x)
            x.cos > Real.0
        } else {
            if x = Real.0 {
                cos_zero
                x.cos = Real.1
                Real.1 > Real.0
                x.cos > Real.0
            } else {
                not Real.0 < x
                not_lt_imp_gte(Real.0, x)
                Real.0 >= x
                x <= Real.0
                x != Real.0
                if not x < Real.0 {
                    not_lt_imp_gte(x, Real.0)
                    x >= Real.0
        square_nonneg(x)
        x * x >= Real.0
        Real.0 <= x * x
                    lte_antisymm(x, Real.0)
                    x = Real.0
                    false
                }
                x < Real.0
                neg_pos_is_neg(x)
                -x > Real.0
                Real.0 < -x
                neg_lt_swap_neg(pi_over_two, x)
                -x < pi_over_two
                cos_pos_on_pos_lt_pi_over_two(-x)
                (-x).cos > Real.0
                cos_neg(-x)
                (-(-x)).cos = (-x).cos
                neg_neg(x)
                -(-x) = x
                x.cos = (-x).cos
                x.cos > Real.0
            }
        }
    }
}

// Sine is positive on (0, pi/2) by the cofunction identity
// x.sin = (pi/2 - x).cos, and on (pi/2, pi) by (pi - x).sin = x.sin.

/// Sine is positive on the open interval (0, pi/2).
theorem sin_pos_on_pos_lt_pi_over_two(x: Real) {
    Real.0 < x and x < pi_over_two implies x.sin > Real.0
} by {
    if Real.0 < x and x < pi_over_two {
        sin_pi_over_two_sub(x)
        x.sin = (pi_over_two - x).cos
        lt_add_right(x, pi_over_two, -x)
        x + -x < pi_over_two + -x
        x + -x = Real.0
        pi_over_two + -x = pi_over_two - x
        Real.0 < pi_over_two - x
        x > Real.0
        neg_pos_is_neg(x)
        -x < Real.0
        lt_add_right(-x, Real.0, pi_over_two)
        -x + pi_over_two < Real.0 + pi_over_two
        -x + pi_over_two = pi_over_two - x
        Real.0 + pi_over_two = pi_over_two
        pi_over_two - x < pi_over_two
        cos_pos_on_pos_lt_pi_over_two(pi_over_two - x)
        (pi_over_two - x).cos > Real.0
        x.sin > Real.0
    }
}

/// The sine of pi minus x is sine of x.
theorem sin_pi_sub(x: Real) {
    (pi - x).sin = x.sin
} by {
    sin_add_pi_over_two(pi_over_two - x)
    (pi_over_two - x + pi_over_two).sin = (pi_over_two - x).cos
    pi_eq_double_pi_over_two
    pi = two * pi_over_two
    pi_over_two - x + pi_over_two = pi - x
    (pi - x).sin = (pi_over_two - x).cos
    sin_pi_over_two_sub(x)
    (pi_over_two - x).cos = x.sin
    (pi - x).sin = x.sin
}

/// Sine is positive on the open interval (0, pi).
theorem sin_pos_on_open_unit_interval(t: Real) {
    Real.0 < t and t < pi implies t.sin > Real.0
} by {
    if Real.0 < t and t < pi {
        if t < pi_over_two {
            sin_pos_on_pos_lt_pi_over_two(t)
            t.sin > Real.0
        } else {
            if t = pi_over_two {
                sin_pi_over_two_one
                t.sin = Real.1
                Real.1 > Real.0
                t.sin > Real.0
            } else {
                not t < pi_over_two
                not_lt_imp_gte(t, pi_over_two)
                t >= pi_over_two
                pi_over_two <= t
                t != pi_over_two
                if not pi_over_two < t {
                    not_lt_imp_gte(pi_over_two, t)
                    pi_over_two >= t
                    t <= pi_over_two
                    lte_antisymm(pi_over_two, t)
                    pi_over_two = t
                    t = pi_over_two
                    false
                }
                pi_over_two < t
                sin_pi_sub(t)
                t.sin = (pi - t).sin
                lt_add_right(t, pi, -t)
                t + -t < pi + -t
                t + -t = Real.0
                pi + -t = pi - t
                Real.0 < pi - t
                lt_add_right(pi_over_two, t, pi_over_two)
                pi_over_two + pi_over_two < t + pi_over_two
                pi_over_two + pi_over_two = pi
                t + pi_over_two = pi_over_two + t
                pi < pi_over_two + t
                lt_add_converse(pi - t, pi_over_two, t)
                (pi - t) + t < pi_over_two + t
                (pi - t) + t = pi
                pi < pi_over_two + t
                pi - t < pi_over_two
                sin_pos_on_pos_lt_pi_over_two(pi - t)
                (pi - t).sin > Real.0
                t.sin > Real.0
            }
        }
    }
}

/// Sine is positive on (0, pi/2].
theorem sin_pos_on_pos_le_pi_over_two(x: Real) {
    Real.0 < x and x <= pi_over_two implies x.sin > Real.0
} by {
    if Real.0 < x and x <= pi_over_two {
        if x < pi_over_two {
            sin_pos_on_pos_lt_pi_over_two(x)
            x.sin > Real.0
        } else {
            not x < pi_over_two
            not_lt_imp_gte(x, pi_over_two)
            x >= pi_over_two
            pi_over_two <= x
            lte_antisymm(x, pi_over_two)
            x = pi_over_two
            sin_pi_over_two_one
            x.sin = Real.1
            Real.1 > Real.0
            x.sin > Real.0
        }
    }
}

// Difference formulas and strict monotonicity.
//
// The key identities are
//     (y - x).sin = y.sin x.cos - y.cos x.sin,
//     tan(y) - tan(x) = (y - x).sin / (x.cos y.cos),
//     y.sin - x.sin = 2 ((x+y)/2).cos ((y-x)/2).sin.
// Together with the positivity of sine and cosine established above, they
// make the tangent strictly increasing on (-pi/2, pi/2) and the sine strictly
// increasing on [-pi/2, pi/2] without any derivative machinery.

/// The sine of a difference.
theorem sin_sub(x: Real, y: Real) {
    (y - x).sin = y.sin * x.cos - y.cos * x.sin
} by {
    sin_add(y, -x)
    (y + -x).sin = y.sin * (-x).cos + y.cos * (-x).sin
    y - x = y + -x
    (y - x).sin = y.sin * (-x).cos + y.cos * (-x).sin
    cos_neg(x)
    (-x).cos = x.cos
    sin_neg(x)
    (-x).sin = -x.sin
    y.sin * (-x).cos + y.cos * (-x).sin = y.sin * x.cos + y.cos * -x.sin
    mul_neg_right(y.cos, x.sin)
    y.cos * -x.sin = -(y.cos * x.sin)
    y.sin * x.cos + y.cos * -x.sin = y.sin * x.cos - y.cos * x.sin
    (y - x).sin = y.sin * x.cos - y.cos * x.sin
}

/// Half the sum of two reals.
define half_sum(x: Real, y: Real) -> Real {
    (x + y) / two
}

/// Half the difference of two reals.
define half_diff(x: Real, y: Real) -> Real {
    (y - x) / two
}

/// Half the sum plus half the difference is the larger point.
theorem half_sum_add_half_diff(x: Real, y: Real) {
    half_sum(x, y) + half_diff(x, y) = y
} by {
    half_sum(x, y) = (x + y) / two
    half_diff(x, y) = (y - x) / two
    two_positive
    two > Real.0
    lt_imp_ne(Real.0, two)
    two != Real.0
    div_add_same_denom(Real.1, x + y, y - x, two)
    Real.1 * (x + y) / two + Real.1 * (y - x) / two =
        Real.1 * ((x + y) + (y - x)) / two
    Real.1 * (x + y) / two + Real.1 * (y - x) / two = ((x + y) + (y - x)) / two
    half_sum(x, y) + half_diff(x, y) = ((x + y) + (y - x)) / two
    (x + y) + (y - x) = two * y
    half_sum(x, y) + half_diff(x, y) = two * y / two
    div_mul_cancel_left(two, y)
    two * y / two = y
    half_sum(x, y) + half_diff(x, y) = y
}

/// Half the sum minus half the difference is the smaller point.
theorem half_sum_sub_half_diff(x: Real, y: Real) {
    half_sum(x, y) - half_diff(x, y) = x
} by {
    half_sum(x, y) = (x + y) / two
    half_diff(x, y) = (y - x) / two
    two_positive
    two > Real.0
    lt_imp_ne(Real.0, two)
    two != Real.0
    div_sub_same_denom(Real.1, Real.1, x + y, y - x, two)
    Real.1 * (x + y) / two - Real.1 * (y - x) / two =
        (Real.1 * (x + y) - Real.1 * (y - x)) / two
    Real.1 * (x + y) / two - Real.1 * (y - x) / two =
        ((x + y) - (y - x)) / two
    half_sum(x, y) - half_diff(x, y) = ((x + y) - (y - x)) / two
    (x + y) - (y - x) = two * x
    half_sum(x, y) - half_diff(x, y) = two * x / two
    div_mul_cancel_left(two, x)
    two * x / two = x
    half_sum(x, y) - half_diff(x, y) = x
}

/// The difference of sines as a product.
theorem sin_sub_sum_to_product(x: Real, y: Real) {
    y.sin - x.sin = two * (half_sum(x, y)).cos * (half_diff(x, y)).sin
} by {
    half_sum_add_half_diff(x, y)
    half_sum(x, y) + half_diff(x, y) = y
    half_sum_sub_half_diff(x, y)
    half_sum(x, y) - half_diff(x, y) = x
    sin_add(half_sum(x, y), half_diff(x, y))
    (half_sum(x, y) + half_diff(x, y)).sin =
        (half_sum(x, y)).sin * (half_diff(x, y)).cos + (half_sum(x, y)).cos * (half_diff(x, y)).sin
    y.sin = (half_sum(x, y)).sin * (half_diff(x, y)).cos + (half_sum(x, y)).cos * (half_diff(x, y)).sin
    sin_sub(half_sum(x, y), half_diff(x, y))
    (half_diff(x, y) - half_sum(x, y)).sin =
        (half_diff(x, y)).sin * (half_sum(x, y)).cos - (half_diff(x, y)).cos * (half_sum(x, y)).sin
    (half_sum(x, y) - half_diff(x, y)).sin =
        (half_sum(x, y)).sin * (half_diff(x, y)).cos - (half_sum(x, y)).cos * (half_diff(x, y)).sin
    x.sin = (half_sum(x, y)).sin * (half_diff(x, y)).cos - (half_sum(x, y)).cos * (half_diff(x, y)).sin
    y.sin - x.sin =
        ((half_sum(x, y)).sin * (half_diff(x, y)).cos + (half_sum(x, y)).cos * (half_diff(x, y)).sin) -
        ((half_sum(x, y)).sin * (half_diff(x, y)).cos - (half_sum(x, y)).cos * (half_diff(x, y)).sin)
    ((half_sum(x, y)).sin * (half_diff(x, y)).cos + (half_sum(x, y)).cos * (half_diff(x, y)).sin) -
        ((half_sum(x, y)).sin * (half_diff(x, y)).cos - (half_sum(x, y)).cos * (half_diff(x, y)).sin) =
        two * (half_sum(x, y)).cos * (half_diff(x, y)).sin
    y.sin - x.sin = two * (half_sum(x, y)).cos * (half_diff(x, y)).sin
}

/// The tangent difference identity.
theorem tan_sub(x: Real, y: Real) {
    x.cos != Real.0 and y.cos != Real.0 implies
    tan(y) - tan(x) = (y - x).sin / (x.cos * y.cos)
} by {
    if x.cos != Real.0 and y.cos != Real.0 {
        tan(y) = y.sin / y.cos
        tan(x) = x.sin / x.cos
        tan(y) - tan(x) = y.sin / y.cos - x.sin / x.cos
        div_mul_cancel_left(x.cos, Real.1)
        x.cos / x.cos = Real.1
        div_mul_cancel_left(y.cos, Real.1)
        y.cos / y.cos = Real.1
        y.sin / y.cos = (y.sin / y.cos) * (x.cos / x.cos)
        (y.sin / y.cos) * (x.cos / x.cos) = (y.sin * x.cos) / (y.cos * x.cos)
        y.sin / y.cos = (y.sin * x.cos) / (y.cos * x.cos)
        x.sin / x.cos = (x.sin / x.cos) * (y.cos / y.cos)
        (x.sin / x.cos) * (y.cos / y.cos) = (x.sin * y.cos) / (x.cos * y.cos)
        x.sin / x.cos = (x.sin * y.cos) / (x.cos * y.cos)
        tan(y) - tan(x) =
            (y.sin * x.cos) / (y.cos * x.cos) - (x.sin * y.cos) / (x.cos * y.cos)
        y.cos * x.cos = x.cos * y.cos
        tan(y) - tan(x) =
            (y.sin * x.cos) / (x.cos * y.cos) - (x.sin * y.cos) / (x.cos * y.cos)
        mul_not_zero(x.cos, y.cos)
        x.cos * y.cos != Real.0
        div_sub_same_denom(Real.1, Real.1, y.sin * x.cos, x.sin * y.cos, x.cos * y.cos)
        Real.1 * (y.sin * x.cos) / (x.cos * y.cos) -
            Real.1 * (x.sin * y.cos) / (x.cos * y.cos) =
            (Real.1 * (y.sin * x.cos) - Real.1 * (x.sin * y.cos)) / (x.cos * y.cos)
        (y.sin * x.cos) / (x.cos * y.cos) - (x.sin * y.cos) / (x.cos * y.cos) =
            (y.sin * x.cos - x.sin * y.cos) / (x.cos * y.cos)
        tan(y) - tan(x) = (y.sin * x.cos - x.sin * y.cos) / (x.cos * y.cos)
        sin_sub(x, y)
        (y - x).sin = y.sin * x.cos - y.cos * x.sin
        y.cos * x.sin = x.sin * y.cos
        (y - x).sin = y.sin * x.cos - x.sin * y.cos
        tan(y) - tan(x) = (y - x).sin / (x.cos * y.cos)
    }
}

/// The tangent is strictly increasing on (-pi/2, pi/2).
theorem tan_strictly_increasing(x: Real, y: Real) {
    -pi_over_two < x and x < y and y < pi_over_two implies tan(x) < tan(y)
} by {
    if -pi_over_two < x and x < y and y < pi_over_two {
        lt_trans(x, y, pi_over_two)
        x < pi_over_two
        cos_pos_on_open_interval(x)
        x.cos > Real.0
        lt_trans(-pi_over_two, x, y)
        -pi_over_two < y
        cos_pos_on_open_interval(y)
        y.cos > Real.0
        Real.0 < x.cos
        Real.0 < y.cos
        lt_imp_ne(Real.0, x.cos)
        x.cos != Real.0
        lt_imp_ne(Real.0, y.cos)
        y.cos != Real.0
        tan_sub(x, y)
        tan(y) - tan(x) = (y - x).sin / (x.cos * y.cos)
        lt_imp_minus_pos(x, y)
        (y - x).is_positive
        pos_gt_zero(y - x)
        y - x > Real.0
        neg_lt_swap_neg(pi_over_two, x)
        -x < pi_over_two
        lt_add_right(y, pi_over_two, -x)
        y + -x < pi_over_two + -x
        lt_add_right(-x, pi_over_two, pi_over_two)
        -x + pi_over_two < pi_over_two + pi_over_two
        pi_over_two + -x = -x + pi_over_two
        lt_trans(y + -x, pi_over_two + -x, pi_over_two + pi_over_two)
        y + -x < pi_over_two + pi_over_two
        pi_eq_double_pi_over_two
        pi_over_two + pi_over_two = pi
        y + -x < pi
        y - x < pi
        sin_pos_on_open_unit_interval(y - x)
        (y - x).sin > Real.0
        mul_pos_pos(x.cos, y.cos)
        Real.0 < x.cos * y.cos
        inverse_of_positive_is_positive(x.cos * y.cos)
        Real.0 < (x.cos * y.cos).inverse
        mul_pos_pos((y - x).sin, (x.cos * y.cos).inverse)
        Real.0 < (y - x).sin * (x.cos * y.cos).inverse
        (y - x).sin / (x.cos * y.cos) = (y - x).sin * (x.cos * y.cos).inverse
        Real.0 < (y - x).sin / (x.cos * y.cos)
        tan(y) - tan(x) > Real.0
        lt_add_right(Real.0, tan(y) - tan(x), tan(x))
        Real.0 + tan(x) < (tan(y) - tan(x)) + tan(x)
        Real.0 + tan(x) = tan(x)
        (tan(y) - tan(x)) + tan(x) = tan(y)
        tan(x) < tan(y)
    }
}

/// The tangent is nondecreasing on (-pi/2, pi/2).
theorem tan_nondecreasing(x: Real, y: Real) {
    -pi_over_two < x and x <= y and y < pi_over_two implies tan(x) <= tan(y)
} by {
    if -pi_over_two < x and x <= y and y < pi_over_two {
        if x < y {
            tan_strictly_increasing(x, y)
            tan(x) < tan(y)
            lt_imp_lte(tan(x), tan(y))
            tan(x) <= tan(y)
        } else {
            not x < y
            not_lt_imp_gte(x, y)
            y <= x
            lte_antisymm(x, y)
            x = y
            y = x
            tan(y) = tan(x)
            lte_refl(tan(x))
            tan(x) <= tan(x)
            tan(x) <= tan(y)
        }
    }
}

// Sine is strictly increasing on [-pi/2, pi/2] via the sum-to-product formula.

/// Pi over two is pi divided by two.
theorem pi_div_two_eq_pi_over_two {
    pi / two = pi_over_two
} by {
    pi = two * pi_over_two
    pi / two = two * pi_over_two / two
    div_mul_cancel_left(two, pi_over_two)
    two * pi_over_two / two = pi_over_two
    pi / two = pi_over_two
}

/// Half the sum of two points of [-pi/2, pi/2] lies strictly inside (-pi/2, pi/2).
theorem half_sum_in_open_interval(x: Real, y: Real) {
    -pi_over_two <= x and x < y and y <= pi_over_two implies
    -pi_over_two < half_sum(x, y) and half_sum(x, y) < pi_over_two
} by {
    if -pi_over_two <= x and x < y and y <= pi_over_two {
        lt_of_lt_of_lte(x, y, pi_over_two)
        x < pi_over_two
        lt_add_right(x, pi_over_two, y)
        x + y < pi_over_two + y
        lte_add_right(y, pi_over_two, pi_over_two)
        y + pi_over_two <= pi_over_two + pi_over_two
        pi_over_two + y = y + pi_over_two
        pi_over_two + y <= pi_over_two + pi_over_two
        lt_of_lt_of_lte(x + y, pi_over_two + y, pi_over_two + pi_over_two)
        x + y < pi_over_two + pi_over_two
        pi_eq_double_pi_over_two
        pi_over_two + pi_over_two = pi
        x + y < pi
        two_positive
        two > Real.0
        div_lt_div_pos(x + y, pi, two)
        (x + y) / two < pi / two
        pi_div_two_eq_pi_over_two
        pi / two = pi_over_two
        (x + y) / two < pi_over_two
        half_sum(x, y) = (x + y) / two
        half_sum(x, y) < pi_over_two

        lt_of_lte_of_lt(-pi_over_two, x, y)
        -pi_over_two < y
        lte_add_right(-pi_over_two, x, y)
        -pi_over_two + y <= x + y
        lt_add_right(-pi_over_two, y, -pi_over_two)
        -pi_over_two + -pi_over_two < y + -pi_over_two
        y + -pi_over_two = -pi_over_two + y
        lt_of_lt_of_lte(-pi_over_two + -pi_over_two, -pi_over_two + y, x + y)
        -pi_over_two + -pi_over_two < x + y
        pi_eq_double_pi_over_two
        pi = pi_over_two + pi_over_two
        neg_distrib(pi_over_two, pi_over_two)
        -(pi_over_two + pi_over_two) = -pi_over_two + -pi_over_two
        -pi_over_two + -pi_over_two = -pi
        -pi < x + y
        two_positive
        two > Real.0
        div_lt_div_pos(-pi, x + y, two)
        (-pi) / two < (x + y) / two
        neg_div(pi, two)
        (-pi) / two = -(pi / two)
        pi_div_two_eq_pi_over_two
        pi / two = pi_over_two
        (-pi) / two = -pi_over_two
        -pi_over_two < (x + y) / two
        half_sum(x, y) = (x + y) / two
        -pi_over_two < half_sum(x, y)
        -pi_over_two < half_sum(x, y) and half_sum(x, y) < pi_over_two
    }
}

/// Half the difference of two points of [-pi/2, pi/2] lies in (0, pi/2].
theorem half_diff_in_interval(x: Real, y: Real) {
    -pi_over_two <= x and x < y and y <= pi_over_two implies
    Real.0 < half_diff(x, y) and half_diff(x, y) <= pi_over_two
} by {
    if -pi_over_two <= x and x < y and y <= pi_over_two {
        lt_imp_minus_pos(x, y)
        (y - x).is_positive
        pos_gt_zero(y - x)
        y - x > Real.0
        two_positive
        two > Real.0
        div_lt_div_pos(Real.0, y - x, two)
        Real.0 / two < (y - x) / two
        Real.0 / two = Real.0
        Real.0 < (y - x) / two
        half_diff(x, y) = (y - x) / two
        Real.0 < half_diff(x, y)

        if not -x <= pi_over_two {
            not_lte_imp_gt(-x, pi_over_two)
            -x > pi_over_two
            pi_over_two < -x
            lt_neg_swap_neg(x, pi_over_two)
            x < -pi_over_two
            false
        }
        -x <= pi_over_two
        add_lte_add(y, pi_over_two, -x, pi_over_two)
        y + -x <= pi_over_two + pi_over_two
        pi_eq_double_pi_over_two
        pi_over_two + pi_over_two = pi
        y + -x <= pi
        y - x <= pi
        inverse_of_positive_is_positive(two)
        Real.0 < two.inverse
        lt_imp_lte(Real.0, two.inverse)
        Real.0 <= two.inverse
        lte_mul_nonneg_right(y - x, pi, two.inverse)
        (y - x) * two.inverse <= pi * two.inverse
        (y - x) / two = (y - x) * two.inverse
        (y - x) / two <= pi * two.inverse
        pi / two = pi * two.inverse
        pi_div_two_eq_pi_over_two
        pi / two = pi_over_two
        (y - x) / two <= pi_over_two
        half_diff(x, y) = (y - x) / two
        half_diff(x, y) <= pi_over_two
        Real.0 < half_diff(x, y) and half_diff(x, y) <= pi_over_two
    }
}

/// Sine is strictly increasing on [-pi/2, pi/2].
theorem sin_strictly_increasing(x: Real, y: Real) {
    -pi_over_two <= x and x < y and y <= pi_over_two implies x.sin < y.sin
} by {
    if -pi_over_two <= x and x < y and y <= pi_over_two {
        half_sum_in_open_interval(x, y)
        -pi_over_two < half_sum(x, y) and half_sum(x, y) < pi_over_two
        -pi_over_two < half_sum(x, y)
        half_sum(x, y) < pi_over_two
        cos_pos_on_open_interval(half_sum(x, y))
        (half_sum(x, y)).cos > Real.0
        half_diff_in_interval(x, y)
        Real.0 < half_diff(x, y) and half_diff(x, y) <= pi_over_two
        Real.0 < half_diff(x, y)
        half_diff(x, y) <= pi_over_two
        sin_pos_on_pos_le_pi_over_two(half_diff(x, y))
        (half_diff(x, y)).sin > Real.0
        sin_sub_sum_to_product(x, y)
        y.sin - x.sin = two * (half_sum(x, y)).cos * (half_diff(x, y)).sin
        two_positive
        Real.0 < two
        mul_pos_pos(two, (half_sum(x, y)).cos)
        Real.0 < two * (half_sum(x, y)).cos
        mul_pos_pos(two * (half_sum(x, y)).cos, (half_diff(x, y)).sin)
        Real.0 < (two * (half_sum(x, y)).cos) * (half_diff(x, y)).sin
        two * (half_sum(x, y)).cos * (half_diff(x, y)).sin =
            (two * (half_sum(x, y)).cos) * (half_diff(x, y)).sin
        Real.0 < two * (half_sum(x, y)).cos * (half_diff(x, y)).sin
        y.sin - x.sin > Real.0
        lt_add_right(Real.0, y.sin - x.sin, x.sin)
        Real.0 + x.sin < (y.sin - x.sin) + x.sin
        Real.0 + x.sin = x.sin
        (y.sin - x.sin) + x.sin = y.sin
        x.sin < y.sin
    }
}

/// Negating both sides of a strict inequality reverses it.
theorem lt_neg_lt(a: Real, b: Real) {
    a < b implies -b < -a
} by {
    if a < b {
        lt_add_right(a, b, -a - b)
        a + (-a - b) < b + (-a - b)
        a + (-a - b) = a + (-a + -b)
        a + (-a + -b) = (a + -a) + -b
        a + -a = Real.0
        (a + -a) + -b = Real.0 + -b
        Real.0 + -b = -b
        a + (-a - b) = -b
        b + (-a - b) = b + (-a + -b)
        -a + -b = -b + -a
        b + (-a + -b) = b + (-b + -a)
        b + (-b + -a) = (b + -b) + -a
        b + -b = Real.0
        (b + -b) + -a = Real.0 + -a
        Real.0 + -a = -a
        b + (-a - b) = -a
        -b < -a
    }
}

// The tangent is unbounded toward the endpoints of (-pi/2, pi/2).
//
// Toward pi/2: Real.sin stays above (pi/4).sin > 0 (strict monotonicity) while Real.cos
// tends to zero (continuity of cosine at pi/2 with (pi/2).cos = 0), so the
// quotient tan = Real.sin/Real.cos grows without bound.  Toward -pi/2 the same holds by
// oddness.

/// The tangent of pi over four is one.
theorem tan_pi_over_four {
    tan(pi_over_four) = Real.1
} by {
    sin_pi_over_two_sub(pi_over_four)
    (pi_over_two - pi_over_four).sin = pi_over_four.cos
    pi_over_two_eq_two_mul_pi_over_four
    pi_over_four + pi_over_four = pi_over_two
    pi_over_two - pi_over_four = pi_over_four
    pi_over_four.sin = pi_over_four.cos
    pi_over_four_pos
    Real.0 < pi_over_four and pi_over_four < pi_over_two
    Real.0 < pi_over_four
    cos_pos_on_pos_lt_pi_over_two(pi_over_four)
    pi_over_four.cos > Real.0
    lt_imp_ne(Real.0, pi_over_four.cos)
    pi_over_four.cos != Real.0
    tan(pi_over_four) = pi_over_four.sin / pi_over_four.cos
    pi_over_four.sin / pi_over_four.cos = pi_over_four.cos / pi_over_four.cos
    div_mul_cancel_left(pi_over_four.cos, Real.1)
    pi_over_four.cos / pi_over_four.cos = Real.1
    tan(pi_over_four) = Real.1
}

/// A nonpositive real is below the tangent of pi over four.
theorem tan_above_nonpos(x: Real) {
    x <= Real.0 implies x < tan(pi_over_four)
} by {
    if x <= Real.0 {
        tan_pi_over_four
        tan(pi_over_four) = Real.1
        lt_of_lte_of_lt(x, Real.0, Real.1)
        x < Real.1
        tan(pi_over_four) = Real.1
        x < tan(pi_over_four)
    }
}

/// The tangent is unbounded above on (0, pi/2) for positive targets.
theorem tan_unbounded_above_pos(x: Real) {
    Real.0 < x implies exists(y: Real) {
        Real.0 < y and y < pi_over_two and x < tan(y)
    }
} by {
    if Real.0 < x {
        pi_over_four_pos
        Real.0 < pi_over_four and pi_over_four < pi_over_two
        Real.0 < pi_over_four
        sin_pos_on_pos_lt_pi_over_two(pi_over_four)
        pi_over_four.sin > Real.0
        Real.0 < pi_over_four.sin
        div_lt_div_pos(Real.0, pi_over_four.sin, x)
        Real.0 / x < pi_over_four.sin / x
        Real.0 / x = Real.0
        Real.0 < pi_over_four.sin / x
        cos_continuous
        continuous(Real.cos)
        continuous_imp_continuous_at(Real.cos, pi_over_two)
        continuous_at(Real.cos, pi_over_two)
        cos_pi_over_two_zero
        pi_over_two.cos = Real.0
        Real.0 < pi_over_four.sin / x
        pi_over_two.cos < pi_over_four.sin / x
        continuous_at_lt_target_neighborhood(Real.cos, pi_over_two, pi_over_four.sin / x)
        let delta: Real satisfy {
            delta.is_positive and forall(y0: Real) {
                y0.is_close(pi_over_two, delta) implies y0.cos < pi_over_four.sin / x
            }
        }
        delta.is_positive
        lt_imp_minus_pos(pi_over_four, pi_over_two)
        (pi_over_two - pi_over_four).is_positive
        pos_gt_zero(pi_over_two - pi_over_four)
        pi_over_two - pi_over_four > Real.0
        eps_smaller_than_both(delta, pi_over_two - pi_over_four)
        let eta: Real satisfy {
            eta.is_positive and eta < delta and eta < pi_over_two - pi_over_four
        }
        eta.is_positive
        eta < delta
        eta < pi_over_two - pi_over_four
        let yw = pi_over_two - eta
        lt_add_right(eta, pi_over_two - pi_over_four, pi_over_four)
        eta + pi_over_four < pi_over_two - pi_over_four + pi_over_four
        pi_over_two - pi_over_four + pi_over_four = pi_over_two
        eta + pi_over_four < pi_over_two
        yw + eta = pi_over_two - eta + eta
        pi_over_two - eta + eta = pi_over_two
        yw + eta = pi_over_two
        pi_over_four + eta = eta + pi_over_four
        pi_over_four + eta < yw + eta
        lt_add_converse(pi_over_four, yw, eta)
        pi_over_four < yw
        lt_trans(Real.0, pi_over_four, yw)
        Real.0 < yw
        neg_pos_is_neg(eta)
        (-eta).is_negative
        neg_lt_zero(-eta)
        -eta < Real.0
        lt_add_right(-eta, Real.0, pi_over_two)
        -eta + pi_over_two < Real.0 + pi_over_two
        -eta + pi_over_two = pi_over_two - eta
        Real.0 + pi_over_two = pi_over_two
        pi_over_two - eta < pi_over_two
        yw < pi_over_two
        lt_add_right(Real.0, delta, pi_over_two)
        Real.0 + pi_over_two < delta + pi_over_two
        Real.0 + pi_over_two = pi_over_two
        delta + pi_over_two = pi_over_two + delta
        pi_over_two < pi_over_two + delta
        lt_trans(yw, pi_over_two, pi_over_two + delta)
        yw < pi_over_two + delta
        lt_neg_lt(eta, delta)
        -delta < -eta
        lt_add_right(-delta, -eta, pi_over_two)
        -delta + pi_over_two < -eta + pi_over_two
        -delta + pi_over_two = pi_over_two - delta
        -eta + pi_over_two = pi_over_two - eta
        pi_over_two - delta < pi_over_two - eta
        yw = pi_over_two - eta
        pi_over_two - delta < yw
        bounds_imp_close(yw, pi_over_two, delta)
        yw.is_close(pi_over_two, delta)
        lt_target_neighborhood_apply(Real.cos, pi_over_two, pi_over_four.sin / x, delta, yw)
        yw.cos < pi_over_four.sin / x
        lt_imp_lte(pi_over_four, yw)
        pi_over_four <= yw
        lt_neg_lt(Real.0, pi_over_two)
        -pi_over_two < -Real.0
        neg_zero
        -Real.0 = Real.0
        -pi_over_two < Real.0
        lt_imp_lte(-pi_over_two, Real.0)
        -pi_over_two <= Real.0
        lt_trans(-pi_over_two, Real.0, pi_over_four)
        -pi_over_two < pi_over_four
        lt_imp_lte(-pi_over_two, pi_over_four)
        -pi_over_two <= pi_over_four
        lt_imp_lte(yw, pi_over_two)
        yw <= pi_over_two
        sin_strictly_increasing(pi_over_four, yw)
        pi_over_four.sin < yw.sin
        lt_imp_lte(pi_over_four.sin, yw.sin)
        pi_over_four.sin <= yw.sin
        lt_imp_ne(Real.0, x)
        x != Real.0
        lt_imp_ne(Real.0, pi_over_four.sin)
        pi_over_four.sin != Real.0
        inverse_div(pi_over_four.sin, x)
        (pi_over_four.sin / x).inverse = x / pi_over_four.sin
        pi_over_four_pos
        Real.0 < pi_over_four and pi_over_four < pi_over_two
        Real.0 < pi_over_four
        pi_over_four < pi_over_two
        cos_pos_on_pos_lt_pi_over_two(yw)
        yw.cos > Real.0
        Real.0 < yw.cos
        lt_imp_ne(Real.0, yw.cos)
        yw.cos != Real.0
        real_inverse_antitone_pos_strict(yw.cos, pi_over_four.sin / x)
        (pi_over_four.sin / x).inverse < yw.cos.inverse
        x / pi_over_four.sin < yw.cos.inverse
        mul_lt_mul_of_pos_right(x / pi_over_four.sin, yw.cos.inverse, pi_over_four.sin)
        (x / pi_over_four.sin) * pi_over_four.sin < yw.cos.inverse * pi_over_four.sin
        mul_div_cancel(x, pi_over_four.sin)
        pi_over_four.sin * (x / pi_over_four.sin) = x
        (x / pi_over_four.sin) * pi_over_four.sin = x
        x < yw.cos.inverse * pi_over_four.sin
        yw.cos.inverse * pi_over_four.sin = pi_over_four.sin * yw.cos.inverse
        x < pi_over_four.sin * yw.cos.inverse
        pi_over_four.sin / yw.cos = pi_over_four.sin * yw.cos.inverse
        x < pi_over_four.sin / yw.cos
        inverse_of_positive_is_positive(yw.cos)
        Real.0 < yw.cos.inverse
        multiply_inequality_with_nonnegative_element(pi_over_four.sin, yw.sin, yw.cos.inverse)
        pi_over_four.sin * yw.cos.inverse <= yw.sin * yw.cos.inverse
        yw.sin / yw.cos = yw.sin * yw.cos.inverse
        pi_over_four.sin * yw.cos.inverse <= yw.sin / yw.cos
        tan(yw) = yw.sin / yw.cos
        pi_over_four.sin / yw.cos <= tan(yw)
        lt_of_lt_of_lte(x, pi_over_four.sin / yw.cos, tan(yw))
        x < tan(yw)
        Real.0 < yw and yw < pi_over_two and x < tan(yw)
        exists(y: Real) {
            Real.0 < y and y < pi_over_two and x < tan(y)
        }
    }
}
/// The tangent is unbounded above on (0, pi/2).
theorem tan_unbounded_above(x: Real) {
    exists(y: Real) {
        Real.0 < y and y < pi_over_two and x < tan(y)
    }
} by {
    if x <= Real.0 {
        tan_above_nonpos(x)
        x < tan(pi_over_four)
        pi_over_four_pos
        Real.0 < pi_over_four and pi_over_four < pi_over_two
        Real.0 < pi_over_four
        pi_over_four < pi_over_two
        Real.0 < pi_over_four and pi_over_four < pi_over_two and x < tan(pi_over_four)
        exists(y0: Real) {
            Real.0 < y0 and y0 < pi_over_two and x < tan(y0)
        }
    } else {
        not x <= Real.0
        not_lte_imp_gt(x, Real.0)
        x > Real.0
        Real.0 < x
        tan_unbounded_above_pos(x)
        let y: Real satisfy {
            Real.0 < y and y < pi_over_two and x < tan(y)
        }
        Real.0 < y and y < pi_over_two and x < tan(y)
        exists(y0: Real) {
            Real.0 < y0 and y0 < pi_over_two and x < tan(y0)
        }
    }
}

/// The tangent is unbounded below on (-pi/2, 0).
theorem tan_unbounded_below(x: Real) {
    exists(y: Real) {
        -pi_over_two < y and y < Real.0 and tan(y) < x
    }
} by {
    tan_unbounded_above(-x)
    let z: Real satisfy {
        Real.0 < z and z < pi_over_two and -x < tan(z)
    }
    Real.0 < z
    z < pi_over_two
    -x < tan(z)
    let y = -z
    neg_pos_is_neg(z)
    z.is_positive
    (-z).is_negative
    neg_lt_zero(-z)
    -z < Real.0
    y < Real.0
    lt_neg_lt(z, pi_over_two)
    -pi_over_two < -z
    -pi_over_two < y
    tan_neg(z)
    tan(-z) = -tan(z)
    neg_lt_swap_neg(x, tan(z))
    -tan(z) < x
    tan(y) = -tan(z)
    tan(y) < x
    -pi_over_two < y and y < Real.0 and tan(y) < x
    exists(witness: Real) {
        witness = y and -pi_over_two < witness and witness < Real.0 and tan(witness) < x
    }
}

// The arctangent.
//
// Following the logarithm in `log.ac`, the arctangent is the default-valued
// witness of the tangent-preimage predicate restricted to the principal branch
// (-pi/2, pi/2).  The tangent is continuous on that interval and unbounded
// toward both endpoints, so the restricted lower preimage {y : tan(y) <= x}
// is nonempty and bounded above strictly below pi/2; its supremum is the
// tangent preimage of x.

/// The tangent agrees with the pointwise quotient of sine by cosine.
theorem tan_eq_pointwise_div {
    tan = pointwise_div_real(Real.sin, Real.cos)
} by {
    forall(x: Real) {
        tan(x) = x.sin / x.cos
        pointwise_div_real_apply(Real.sin, Real.cos, x)
        pointwise_div_real(Real.sin, Real.cos, x) = x.sin / x.cos
        tan(x) = pointwise_div_real(Real.sin, Real.cos, x)
    }
    function_extensionality(tan, pointwise_div_real(Real.sin, Real.cos))
}

/// The tangent is continuous at every point of (-pi/2, pi/2).
theorem tan_continuous_at(x0: Real) {
    -pi_over_two < x0 and x0 < pi_over_two implies continuous_at(tan, x0)
} by {
    if -pi_over_two < x0 and x0 < pi_over_two {
        cos_pos_on_open_interval(x0)
        x0.cos > Real.0
        lt_imp_ne(Real.0, x0.cos)
        x0.cos != Real.0
        sin_continuous
        continuous(Real.sin)
        continuous_imp_continuous_at(Real.sin, x0)
        continuous_at(Real.sin, x0)
        cos_continuous
        continuous(Real.cos)
        continuous_imp_continuous_at(Real.cos, x0)
        continuous_at(Real.cos, x0)
        continuous_at_pointwise_div_real(Real.sin, Real.cos, x0)
        continuous_at(pointwise_div_real(Real.sin, Real.cos), x0)
        tan_eq_pointwise_div
        tan = pointwise_div_real(Real.sin, Real.cos)
        function_eq_transport_predicate_rev(
            function(h: Real -> Real) { continuous_at(h, x0) },
            tan,
            pointwise_div_real(Real.sin, Real.cos)
        )
        continuous_at(tan, x0)
    }
}

/// True if y lies in the lower tangent preimage of x within the principal branch.
define tan_lower_preimage_contains(x: Real, y: Real) -> Bool {
    -pi_over_two < y and y < pi_over_two and tan(y) <= x
}

/// The set of y in (-pi/2, pi/2) with tan(y) <= x.
define tan_lower_preimage(x: Real) -> Set[Real] {
    Set[Real].new(tan_lower_preimage_contains(x))
}

/// Membership in the lower tangent preimage is the defining conjunction.
theorem tan_lower_preimage_contains_eq(x: Real, y: Real) {
    tan_lower_preimage(x).contains(y) = tan_lower_preimage_contains(x, y)
} by {
}

/// The lower tangent preimage of every real is nonempty.
theorem tan_lower_preimage_nonempty(x: Real) {
    is_nonempty(tan_lower_preimage(x))
} by {
    tan_unbounded_below(x)
    let y: Real satisfy {
        -pi_over_two < y and y < Real.0 and tan(y) < x
    }
    -pi_over_two < y
    y < Real.0
    tan(y) < x
    lt_imp_lte(tan(y), x)
    tan(y) <= x
    lt_trans(y, Real.0, pi_over_two)
    y < pi_over_two
    tan_lower_preimage_contains(x, y)
    tan_lower_preimage_contains_eq(x, y)
    tan_lower_preimage(x).contains(y)
    is_nonempty(tan_lower_preimage(x))
}

/// The lower tangent preimage of every real is bounded above strictly below pi/2.
theorem tan_lower_preimage_upper_bound_below(x: Real) {
    exists(b: Real) {
        is_set_upper_bound(tan_lower_preimage(x), b) and b < pi_over_two
    }
} by {
    tan_unbounded_above(x)
    let z: Real satisfy {
        Real.0 < z and z < pi_over_two and x < tan(z)
    }
    Real.0 < z
    z < pi_over_two
    x < tan(z)
    forall(y: Real) {
        if tan_lower_preimage(x).contains(y) {
            tan_lower_preimage_contains_eq(x, y)
            tan_lower_preimage_contains(x, y)
            tan_lower_preimage_contains(x, y) =
                (-pi_over_two < y and y < pi_over_two and tan(y) <= x)
            -pi_over_two < y
            y < pi_over_two
            tan(y) <= x
            if not y < z {
                not_lt_imp_gte(y, z)
                z <= y
                lt_neg_lt(Real.0, pi_over_two)
                -pi_over_two < -Real.0
                neg_zero
                -Real.0 = Real.0
                -pi_over_two < Real.0
                lt_trans(-pi_over_two, Real.0, z)
                -pi_over_two < z
                tan_nondecreasing(z, y)
                tan(z) <= tan(y)
                lte_trans(tan(z), tan(y), x)
                tan(z) <= x
                x < tan(z)
                false
            }
            y < z
            lt_imp_lte(y, z)
            y <= z
        }
    }
    is_set_upper_bound(tan_lower_preimage(x), z)
    z < pi_over_two
    is_set_upper_bound(tan_lower_preimage(x), z) and z < pi_over_two
    exists(b: Real) {
        is_set_upper_bound(tan_lower_preimage(x), b) and b < pi_over_two
    }
}

/// Every real number is a tangent value in the principal branch.
theorem exists_tan_of(x: Real) {
    exists(y: Real) {
        tan(y) = x and -pi_over_two < y and y < pi_over_two
    }
} by {
    let s = tan_lower_preimage(x)
    tan_lower_preimage_nonempty(x)
    is_nonempty(s)
    tan_lower_preimage_upper_bound_below(x)
    exists(b: Real) {
        is_set_upper_bound(s, b) and b < pi_over_two
    }
    let b: Real satisfy {
        is_set_upper_bound(s, b) and b < pi_over_two
    }
    is_set_upper_bound(s, b)
    b < pi_over_two
    completeness(s)
    let y: Real satisfy {
        is_set_supremum(s, y)
    }
    is_set_supremum(s, y)
    set_supremum_le_upper_bound(s, y, b)
    y <= b
    lt_of_lte_of_lt(y, b, pi_over_two)
    y < pi_over_two
    is_nonempty(s) = exists(x0: Real) { s.contains(x0) }
    let y0: Real satisfy {
        s.contains(y0)
    }
    s.contains(y0)
    tan_lower_preimage_contains_eq(x, y0)
    tan_lower_preimage_contains(x, y0)
    tan_lower_preimage_contains(x, y0) =
        (-pi_over_two < y0 and y0 < pi_over_two and tan(y0) <= x)
    -pi_over_two < y0
    set_member_le_supremum(s, y, y0)
    y0 <= y
        lt_of_lt_of_lte(-pi_over_two, y0, y)
    -pi_over_two < y
    if tan(y) < x {
        tan_continuous_at(y)
        continuous_at(tan, y)
        continuous_at_lt_target_neighborhood(tan, y, x)
        let delta: Real satisfy {
            delta.is_positive and forall(z: Real) {
                z.is_close(y, delta) implies tan(z) < x
            }
        }
        delta.is_positive
        lt_imp_minus_pos(y, pi_over_two)
        (pi_over_two - y).is_positive
        pos_gt_zero(pi_over_two - y)
        pi_over_two - y > Real.0
        eps_smaller_than_both(delta, pi_over_two - y)
        let eta: Real satisfy {
            eta.is_positive and eta < delta and eta < pi_over_two - y
        }
        eta.is_positive
        eta < delta
        eta < pi_over_two - y
        let z = y + eta
        lt_add_right(eta, pi_over_two - y, y)
        eta + y < pi_over_two - y + y
        pi_over_two - y + y = pi_over_two
        eta + y < pi_over_two
        y + eta < pi_over_two
        z < pi_over_two
        lt_add_pos(y, eta)
        y < y + eta
        y < z
        lt_add_right(-pi_over_two, y, pi_over_two)
        -pi_over_two + pi_over_two < y + pi_over_two
        -pi_over_two + pi_over_two = Real.0
        Real.0 < y + pi_over_two
        add_lt_lt(Real.0, eta, Real.0, y + pi_over_two)
        Real.0 + Real.0 < eta + (y + pi_over_two)
        Real.0 + Real.0 = Real.0
        eta + (y + pi_over_two) = y + eta + pi_over_two
        Real.0 < y + eta + pi_over_two
        lt_add_right(Real.0, y + eta + pi_over_two, -pi_over_two)
        Real.0 + -pi_over_two < (y + eta + pi_over_two) + -pi_over_two
        Real.0 + -pi_over_two = -pi_over_two
        (y + eta + pi_over_two) + -pi_over_two = y + eta
        -pi_over_two < y + eta
        -pi_over_two < z
        lt_add_right(Real.0, delta, y)
        Real.0 + y < delta + y
        Real.0 + y = y
        delta + y = y + delta
        y < y + delta
        lt_trans(y, z, y + delta)
        z < y + delta
        lt_add_right(Real.0, eta, y - delta)
        Real.0 + (y - delta) < eta + (y - delta)
        Real.0 + (y - delta) = y - delta
        y - delta < y + eta - delta
        y + eta - delta = y - delta + eta
        y - delta < z - delta
        pos_gt_zero(delta)
        delta > Real.0
        neg_pos_is_neg(delta)
        (-delta).is_negative
        neg_lt_zero(-delta)
        -delta < Real.0
        lt_add_right(-delta, Real.0, z)
        -delta + z < Real.0 + z
        -delta + z = z - delta
        Real.0 + z = z
        z - delta < z
        lt_trans(y - delta, z - delta, z)
        y - delta < z
        bounds_imp_close(z, y, delta)
        z.is_close(y, delta)
        lt_target_neighborhood_apply(tan, y, x, delta, z)
        tan(z) < x
        lt_imp_lte(tan(z), x)
        tan(z) <= x
        -pi_over_two < z
        z < pi_over_two
        tan_lower_preimage_contains(x, z)
        tan_lower_preimage_contains_eq(x, z)
        s.contains(z)
        set_member_le_supremum(s, y, z)
        z <= y
        y < z
        false
    }
    if x < tan(y) {
        tan_continuous_at(y)
        continuous_at(tan, y)
        continuous_at_target_lt_neighborhood(tan, y, x)
        let delta2: Real satisfy {
            delta2.is_positive and forall(z: Real) {
                z.is_close(y, delta2) implies x < tan(z)
            }
        }
        delta2.is_positive
        lt_imp_minus_pos(-pi_over_two, y)
        (y - -pi_over_two).is_positive
        pos_gt_zero(y - -pi_over_two)
        y - -pi_over_two > Real.0
        y + pi_over_two > Real.0
        eps_smaller_than_both(delta2, y + pi_over_two)
        let eta2: Real satisfy {
            eta2.is_positive and eta2 < delta2 and eta2 < y + pi_over_two
        }
        eta2.is_positive
        eta2 < delta2
        eta2 < y + pi_over_two
        let z2 = y - eta2
        lt_add_right(eta2, y + pi_over_two, -pi_over_two - eta2)
        eta2 + (-pi_over_two - eta2) < (y + pi_over_two) + (-pi_over_two - eta2)
        eta2 + (-pi_over_two - eta2) = eta2 + (-pi_over_two + -eta2)
        -pi_over_two + -eta2 = -eta2 + -pi_over_two
        eta2 + (-pi_over_two + -eta2) = eta2 + (-eta2 + -pi_over_two)
        eta2 + (-eta2 + -pi_over_two) = (eta2 + -eta2) + -pi_over_two
        eta2 + -eta2 = Real.0
        (eta2 + -eta2) + -pi_over_two = Real.0 + -pi_over_two
        Real.0 + -pi_over_two = -pi_over_two
        eta2 + (-pi_over_two - eta2) = -pi_over_two
        -pi_over_two - eta2 = -pi_over_two + -eta2
        (y + pi_over_two) + (-pi_over_two + -eta2) = y + (pi_over_two + (-pi_over_two + -eta2))
        pi_over_two + (-pi_over_two + -eta2) = (pi_over_two + -pi_over_two) + -eta2
        pi_over_two + -pi_over_two = Real.0
        (pi_over_two + -pi_over_two) + -eta2 = Real.0 + -eta2
        Real.0 + -eta2 = -eta2
        y + (pi_over_two + (-pi_over_two + -eta2)) = y + -eta2
        y + -eta2 = y - eta2
        (y + pi_over_two) + (-pi_over_two - eta2) = y - eta2
        -pi_over_two < y - eta2
        z2 > -pi_over_two
        -pi_over_two < z2
        neg_pos_is_neg(eta2)
        (-eta2).is_negative
        neg_lt_zero(-eta2)
        -eta2 < Real.0
        lt_add_right(-eta2, Real.0, y)
        -eta2 + y < Real.0 + y
        -eta2 + y = y - eta2
        Real.0 + y = y
        y - eta2 < y
        z2 < y
        z2 < y
        lt_trans(z2, y, pi_over_two)
        z2 < pi_over_two
        lt_add_right(Real.0, delta2, y)
        Real.0 + y < delta2 + y
        Real.0 + y = y
        delta2 + y = y + delta2
        y < y + delta2
        z2 < y
        lt_trans(z2, y, y + delta2)
        z2 < y + delta2
        lt_neg_lt(eta2, delta2)
        -delta2 < -eta2
        lt_add_right(-delta2, -eta2, y)
        -delta2 + y < -eta2 + y
        -delta2 + y = y - delta2
        -eta2 + y = y - eta2
        y - delta2 < y - eta2
        y - delta2 < z2
        bounds_imp_close(z2, y, delta2)
        z2.is_close(y, delta2)
        target_lt_neighborhood_apply(tan, y, x, delta2, z2)
        x < tan(z2)
        forall(w: Real) {
            if tan_lower_preimage(x).contains(w) {
                tan_lower_preimage_contains_eq(x, w)
                tan_lower_preimage_contains(x, w)
                tan_lower_preimage_contains(x, w) =
                    (-pi_over_two < w and w < pi_over_two and tan(w) <= x)
                -pi_over_two < w
                w < pi_over_two
                tan(w) <= x
                if not w < z2 {
                    not_lt_imp_gte(w, z2)
                    z2 <= w
                    -pi_over_two < z2
                    tan_nondecreasing(z2, w)
                    tan(z2) <= tan(w)
                    lte_trans(tan(z2), tan(w), x)
                    tan(z2) <= x
                    x < tan(z2)
                    false
                }
                w < z2
                lt_imp_lte(w, z2)
                w <= z2
            }
        }
        is_set_upper_bound(s, z2)
        set_supremum_le_upper_bound(s, y, z2)
        y <= z2
        z2 < y
        false
    }
    not_lt_imp_gte(tan(y), x)
    tan(y) >= x
    not_lt_imp_gte(x, tan(y))
    x >= tan(y)
    tan(y) <= x
    lte_antisymm(tan(y), x)
    tan(y) = x
    tan(y) = x and -pi_over_two < y and y < pi_over_two
    exists(z0: Real) {
        tan(z0) = x and -pi_over_two < z0 and z0 < pi_over_two
    }
}

/// True if y is an arctangent of x: a tangent preimage in the principal branch.
define is_arctan(x: Real, y: Real) -> Bool {
    tan(y) = x and -pi_over_two < y and y < pi_over_two
}

/// The arctangent: the unique y in (-pi/2, pi/2) with tan(y) = x.
define arctan(x: Real) -> Real {
    choose_or_default(is_arctan(x), Real.0)
}

/// The arctangent is a tangent preimage in the principal branch.
theorem is_arctan_spec(x: Real) {
    is_arctan(x, arctan(x))
} by {
    exists_tan_of(x)
    let y: Real satisfy {
        tan(y) = x and -pi_over_two < y and y < pi_over_two
    }
    tan(y) = x
    -pi_over_two < y
    y < pi_over_two
    is_arctan(x, y)
    exists(y0: Real) { is_arctan(x, y0) }
    choose_or_default_spec(is_arctan(x), Real.0)
    is_arctan(x, choose_or_default(is_arctan(x), Real.0))
    arctan(x) = choose_or_default(is_arctan(x), Real.0)
    is_arctan(x, arctan(x))
}

/// Tangent preimages in the principal branch are unique.
theorem tan_injective_on_open_interval(x: Real, y: Real) {
    -pi_over_two < x and x < pi_over_two and -pi_over_two < y and y < pi_over_two and
    tan(x) = tan(y) implies x = y
} by {
    if -pi_over_two < x and x < pi_over_two and -pi_over_two < y and y < pi_over_two and
       tan(x) = tan(y) {
        if x < y {
            tan_strictly_increasing(x, y)
            tan(x) < tan(y)
            false
        }
        if y < x {
            tan_strictly_increasing(y, x)
            tan(y) < tan(x)
            false
        }
        not x < y
        not_lt_imp_gte(x, y)
        y <= x
        not y < x
        not_lt_imp_gte(y, x)
        x <= y
        lte_antisymm(x, y)
        x = y
    }
}

/// The arctangent is the unique principal-branch tangent preimage.
theorem is_arctan_unique(x: Real, y: Real) {
    is_arctan(x, y) implies arctan(x) = y
} by {
    if is_arctan(x, y) {
        is_arctan(x, y) = (tan(y) = x and -pi_over_two < y and y < pi_over_two)
        tan(y) = x
        -pi_over_two < y
        y < pi_over_two
        forall(z: Real) {
            if is_arctan(x, z) {
                is_arctan(x, z) = (tan(z) = x and -pi_over_two < z and z < pi_over_two)
                tan(z) = x
                tan(z) = tan(y)
                -pi_over_two < z
                z < pi_over_two
                tan_injective_on_open_interval(y, z)
                y = z
                z = y
            }
        }
        exists_unique_intro(is_arctan(x), y)
        exists_unique(is_arctan(x))
        choose_or_default_unique_eq(is_arctan(x), Real.0, y)
        arctan(x) = y
    }
}

/// Tangent of the arctangent is the identity.
theorem tan_arctan(x: Real) {
    tan(arctan(x)) = x
} by {
    is_arctan_spec(x)
    is_arctan(x, arctan(x))
    is_arctan(x, arctan(x)) =
        (tan(arctan(x)) = x and -pi_over_two < arctan(x) and arctan(x) < pi_over_two)
    tan(arctan(x)) = x
}

/// The arctangent of a tangent value in the principal branch is the argument.
theorem arctan_tan(x: Real) {
    -pi_over_two < x and x < pi_over_two implies arctan(tan(x)) = x
} by {
    if -pi_over_two < x and x < pi_over_two {
        is_arctan(tan(x), x)
        is_arctan_unique(tan(x), x)
        arctan(tan(x)) = x
    }
}

/// The arctangent is odd.
theorem arctan_neg(x: Real) {
    arctan(-x) = -arctan(x)
} by {
    tan_arctan(x)
    tan(arctan(x)) = x
    tan_neg(arctan(x))
    tan(-arctan(x)) = -tan(arctan(x))
    tan(-arctan(x)) = -x
    is_arctan_spec(x)
    is_arctan(x, arctan(x))
    is_arctan(x, arctan(x)) =
        (tan(arctan(x)) = x and -pi_over_two < arctan(x) and arctan(x) < pi_over_two)
    -pi_over_two < arctan(x)
    arctan(x) < pi_over_two
    lt_neg_lt(arctan(x), pi_over_two)
    -pi_over_two < -arctan(x)
    lt_neg_lt(-pi_over_two, arctan(x))
    -arctan(x) < pi_over_two
    is_arctan(-x, -arctan(x))
    is_arctan_unique(-x, -arctan(x))
    arctan(-x) = -arctan(x)
}

/// The arctangent of zero is zero.
theorem arctan_zero {
    arctan(Real.0) = Real.0
} by {
    tan_zero
    tan(Real.0) = Real.0
    pi_over_two_pos
    Real.0 < pi_over_two
    lt_neg_lt(Real.0, pi_over_two)
    -pi_over_two < -Real.0
    neg_zero
    -Real.0 = Real.0
    -pi_over_two < Real.0
    is_arctan(Real.0, Real.0)
    is_arctan_unique(Real.0, Real.0)
    arctan(Real.0) = Real.0
}

/// The arctangent of one is pi over four.
theorem arctan_one {
    arctan(Real.1) = pi_over_four
} by {
    tan_pi_over_four
    tan(pi_over_four) = Real.1
    pi_over_four_pos
    Real.0 < pi_over_four and pi_over_four < pi_over_two
    Real.0 < pi_over_four
    pi_over_four < pi_over_two
    pi_over_two_pos
    Real.0 < pi_over_two
    lt_neg_lt(Real.0, pi_over_two)
    -pi_over_two < -Real.0
    neg_zero
    -Real.0 = Real.0
    -pi_over_two < Real.0
    lt_trans(-pi_over_two, Real.0, pi_over_four)
    -pi_over_two < pi_over_four
    is_arctan(Real.1, pi_over_four)
    is_arctan_unique(Real.1, pi_over_four)
    arctan(Real.1) = pi_over_four
}

// The arcsine.
//
// Sine is strictly increasing on [-pi/2, pi/2] (proved above) and maps the
// endpoints to -1 and 1, so by the intermediate value theorem every point of
// [-1, 1] is a sine value on the principal branch.  The arcsine is the
// default-valued witness of the restricted sine-preimage predicate.

/// True if y is an arcsine of x: a sine preimage in the principal branch.
define is_arcsin(x: Real, y: Real) -> Bool {
    y.sin = x and -pi_over_two <= y and y <= pi_over_two
}

/// The arcsine: the unique y in [-pi/2, pi/2] with y.sin = x.
define arcsin(x: Real) -> Real {
    choose_or_default(is_arcsin(x), Real.0)
}

/// Sine preimages in the principal branch are unique.
theorem sin_injective_on_closed_interval(x: Real, y: Real) {
    -pi_over_two <= x and x <= pi_over_two and -pi_over_two <= y and y <= pi_over_two and
    x.sin = y.sin implies x = y
} by {
    if -pi_over_two <= x and x <= pi_over_two and -pi_over_two <= y and y <= pi_over_two and
       x.sin = y.sin {
        if x < y {
            sin_strictly_increasing(x, y)
            x.sin < y.sin
            false
        }
        if y < x {
            sin_strictly_increasing(y, x)
            y.sin < x.sin
            false
        }
        not x < y
        not_lt_imp_gte(x, y)
        y <= x
        not y < x
        not_lt_imp_gte(y, x)
        x <= y
        lte_antisymm(x, y)
        x = y
    }
}

/// Every point of [-1, 1] is a sine value on the principal branch.
theorem exists_arcsin(x: Real) {
    -Real.1 <= x and x <= Real.1 implies exists(y: Real) {
        y.sin = x and -pi_over_two <= y and y <= pi_over_two
    }
} by {
    if -Real.1 <= x and x <= Real.1 {
        sin_neg(pi_over_two)
        (-pi_over_two).sin = -pi_over_two.sin
        sin_pi_over_two_one
        pi_over_two.sin = Real.1
        (-pi_over_two).sin = -Real.1
        sin_continuous
        continuous(Real.sin)
        pi_over_two_pos
        Real.0 < pi_over_two
        lt_neg_lt(Real.0, pi_over_two)
        -pi_over_two < -Real.0
        neg_zero
        -Real.0 = Real.0
        -pi_over_two < Real.0
        lt_imp_lte(-pi_over_two, Real.0)
        -pi_over_two <= Real.0
        lt_imp_lte(Real.0, pi_over_two)
        Real.0 <= pi_over_two
        lte_trans(-pi_over_two, Real.0, pi_over_two)
        -pi_over_two <= pi_over_two
        lte_refl(pi_over_two)
        pi_over_two <= pi_over_two
        lte_trans(-pi_over_two, Real.0, pi_over_two)
        -pi_over_two <= pi_over_two
        (-pi_over_two).sin = -Real.1
        -Real.1 <= x
        (-pi_over_two).sin <= x
        sin_pi_over_two_one
        pi_over_two.sin = Real.1
        x <= Real.1
        x <= pi_over_two.sin
        intermediate_value_closed_interval(Real.sin, -pi_over_two, pi_over_two, x)
        let yw: Real satisfy {
            closed_interval_set(-pi_over_two, pi_over_two).contains(yw) and yw.sin = x
        }
        closed_interval_set(-pi_over_two, pi_over_two).contains(yw)
        yw.sin = x
        closed_interval_set_contains_eq(-pi_over_two, pi_over_two, yw)
        closed_interval_set(-pi_over_two, pi_over_two).contains(yw) =
            closed_interval(-pi_over_two, pi_over_two, yw)
        closed_interval(-pi_over_two, pi_over_two, yw) =
            (-pi_over_two <= yw and yw <= pi_over_two)
        -pi_over_two <= yw
        yw <= pi_over_two
        yw.sin = x and -pi_over_two <= yw and yw <= pi_over_two
        exists(y: Real) {
            y.sin = x and -pi_over_two <= y and y <= pi_over_two
        }
    }
}

/// The arcsine is a sine preimage in the principal branch.
theorem is_arcsin_spec(x: Real) {
    -Real.1 <= x and x <= Real.1 implies is_arcsin(x, arcsin(x))
} by {
    if -Real.1 <= x and x <= Real.1 {
        exists_arcsin(x)
        let y: Real satisfy {
            y.sin = x and -pi_over_two <= y and y <= pi_over_two
        }
        y.sin = x
        -pi_over_two <= y
        y <= pi_over_two
        is_arcsin(x, y)
        exists(y0: Real) { is_arcsin(x, y0) }
        choose_or_default_spec(is_arcsin(x), Real.0)
        is_arcsin(x, choose_or_default(is_arcsin(x), Real.0))
        arcsin(x) = choose_or_default(is_arcsin(x), Real.0)
        is_arcsin(x, arcsin(x))
    }
}

/// The arcsine is the unique principal-branch sine preimage.
theorem is_arcsin_unique(x: Real, y: Real) {
    is_arcsin(x, y) implies arcsin(x) = y
} by {
    if is_arcsin(x, y) {
        is_arcsin(x, y) = (y.sin = x and -pi_over_two <= y and y <= pi_over_two)
        y.sin = x
        -pi_over_two <= y
        y <= pi_over_two
        forall(z: Real) {
            if is_arcsin(x, z) {
                is_arcsin(x, z) = (z.sin = x and -pi_over_two <= z and z <= pi_over_two)
                z.sin = x
                z.sin = y.sin
                -pi_over_two <= z
                z <= pi_over_two
                sin_injective_on_closed_interval(y, z)
                y = z
                z = y
            }
        }
        exists_unique_intro(is_arcsin(x), y)
        exists_unique(is_arcsin(x))
        choose_or_default_unique_eq(is_arcsin(x), Real.0, y)
        arcsin(x) = y
    }
}

/// Sine of the arcsine is the identity on [-1, 1].
theorem sin_arcsin(x: Real) {
    -Real.1 <= x and x <= Real.1 implies (arcsin(x)).sin = x
} by {
    if -Real.1 <= x and x <= Real.1 {
        is_arcsin_spec(x)
        is_arcsin(x, arcsin(x))
        is_arcsin(x, arcsin(x)) =
            ((arcsin(x)).sin = x and -pi_over_two <= arcsin(x) and arcsin(x) <= pi_over_two)
        (arcsin(x)).sin = x
    }
}

/// The arcsine of a sine value in the principal branch is the argument.
theorem arcsin_sin(x: Real) {
    -pi_over_two <= x and x <= pi_over_two implies arcsin(x.sin) = x
} by {
    if -pi_over_two <= x and x <= pi_over_two {
        is_arcsin(x.sin, x)
        is_arcsin_unique(x.sin, x)
        arcsin(x.sin) = x
    }
}

/// The arcsine is odd on [-1, 1].
theorem arcsin_neg(x: Real) {
    -Real.1 <= x and x <= Real.1 implies arcsin(-x) = -arcsin(x)
} by {
    if -Real.1 <= x and x <= Real.1 {
        sin_arcsin(x)
        (arcsin(x)).sin = x
        sin_neg(arcsin(x))
        (-arcsin(x)).sin = -(arcsin(x)).sin
        (-arcsin(x)).sin = -x
        is_arcsin_spec(x)
        is_arcsin(x, arcsin(x))
        is_arcsin(x, arcsin(x)) =
            ((arcsin(x)).sin = x and -pi_over_two <= arcsin(x) and arcsin(x) <= pi_over_two)
        -pi_over_two <= arcsin(x)
        arcsin(x) <= pi_over_two
        // -arcsin(x) lies in [-pi/2, pi/2]
        // negating the interval membership of arcsin(x)
        if not -pi_over_two <= -arcsin(x) {
            not_lte_imp_gt(-pi_over_two, -arcsin(x))
            -pi_over_two > -arcsin(x)
            -arcsin(x) < -pi_over_two
            lt_neg_swap_neg(-arcsin(x), pi_over_two)
            pi_over_two < -(-arcsin(x))
            neg_neg(arcsin(x))
            -(-arcsin(x)) = arcsin(x)
            pi_over_two < arcsin(x)
            false
        }
        -pi_over_two <= -arcsin(x)
        -pi_over_two <= -arcsin(x)
        if not -arcsin(x) <= pi_over_two {
            not_lte_imp_gt(-arcsin(x), pi_over_two)
            -arcsin(x) > pi_over_two
            pi_over_two < -arcsin(x)
            lt_neg_swap_neg(pi_over_two, arcsin(x))
            arcsin(x) < -pi_over_two
            false
        }
        -arcsin(x) <= pi_over_two
        -arcsin(x) <= pi_over_two
        is_arcsin(-x, -arcsin(x))
        is_arcsin_unique(-x, -arcsin(x))
        arcsin(-x) = -arcsin(x)
    }
}

/// The arcsine of zero is zero.
theorem arcsin_zero {
    arcsin(Real.0) = Real.0
} by {
    sin_zero
    (Real.0).sin = Real.0
    pi_over_two_pos
    Real.0 < pi_over_two
    lt_neg_lt(Real.0, pi_over_two)
    -pi_over_two < -Real.0
    neg_zero
    -Real.0 = Real.0
    -pi_over_two < Real.0
    lt_imp_lte(-pi_over_two, Real.0)
    -pi_over_two <= Real.0
    lt_imp_lte(Real.0, pi_over_two)
        Real.0 <= pi_over_two
        lte_trans(-pi_over_two, Real.0, pi_over_two)
        -pi_over_two <= pi_over_two
        lte_refl(pi_over_two)
        pi_over_two <= pi_over_two
    is_arcsin(Real.0, Real.0)
    is_arcsin_unique(Real.0, Real.0)
    arcsin(Real.0) = Real.0
}

/// The arcsine of one is pi over two.
theorem arcsin_one {
    arcsin(Real.1) = pi_over_two
} by {
    sin_pi_over_two_one
    pi_over_two.sin = Real.1
    pi_over_two_pos
    Real.0 < pi_over_two
    lt_neg_lt(Real.0, pi_over_two)
    -pi_over_two < -Real.0
    neg_zero
    -Real.0 = Real.0
    -pi_over_two < Real.0
    lt_imp_lte(-pi_over_two, Real.0)
    -pi_over_two <= Real.0
    lt_imp_lte(Real.0, pi_over_two)
        Real.0 <= pi_over_two
        lte_trans(-pi_over_two, Real.0, pi_over_two)
        -pi_over_two <= pi_over_two
        lte_refl(pi_over_two)
        pi_over_two <= pi_over_two
    is_arcsin(Real.1, pi_over_two)
    is_arcsin_unique(Real.1, pi_over_two)
    arcsin(Real.1) = pi_over_two
}

/// The arcsine of negative one is negative pi over two.
theorem arcsin_neg_one {
    arcsin(-Real.1) = -pi_over_two
} by {
    sin_neg(pi_over_two)
    (-pi_over_two).sin = -pi_over_two.sin
    sin_pi_over_two_one
    pi_over_two.sin = Real.1
    (-pi_over_two).sin = -Real.1
    pi_over_two_pos
    Real.0 < pi_over_two
    lt_neg_lt(Real.0, pi_over_two)
    -pi_over_two < -Real.0
    neg_zero
    -Real.0 = Real.0
    -pi_over_two < Real.0
    lt_imp_lte(-pi_over_two, Real.0)
    -pi_over_two <= Real.0
    lt_imp_lte(Real.0, pi_over_two)
        Real.0 <= pi_over_two
        lte_trans(-pi_over_two, Real.0, pi_over_two)
        -pi_over_two <= pi_over_two
        lte_refl(pi_over_two)
        pi_over_two <= pi_over_two
    is_arcsin(-Real.1, -pi_over_two)
    is_arcsin_unique(-Real.1, -pi_over_two)
    arcsin(-Real.1) = -pi_over_two
}

// The arccosine and the bridge identities.
//
// The arccosine is pi/2 minus the arcsine, so the cofunction identities give
// (arccos x).cos = x on [-1, 1] and arccos(Real.cos x) = x on [0, pi].  The bridge
// identities express (arcsin x).cos = (arccos x).sin as (1 - x^2).sqrt.

/// The arccosine: pi over two minus the arcsine.
define arccos(x: Real) -> Real {
    pi_over_two - arcsin(x)
}

/// The arccosine plus the arcsine is pi over two.
theorem arccos_add_arcsin(x: Real) {
    arccos(x) + arcsin(x) = pi_over_two
} by {
    arccos(x) = pi_over_two - arcsin(x)
    arccos(x) + arcsin(x) = pi_over_two - arcsin(x) + arcsin(x)
    pi_over_two - arcsin(x) + arcsin(x) = pi_over_two
    arccos(x) + arcsin(x) = pi_over_two
}

/// Cosine of the arccosine is the identity on [-1, 1].
theorem cos_arccos(x: Real) {
    -Real.1 <= x and x <= Real.1 implies (arccos(x)).cos = x
} by {
    if -Real.1 <= x and x <= Real.1 {
        arccos(x) = pi_over_two - arcsin(x)
        cos_pi_over_two_sub(arcsin(x))
        (pi_over_two - arcsin(x)).cos = (arcsin(x)).sin
        (arccos(x)).cos = (arcsin(x)).sin
        sin_arcsin(x)
        (arcsin(x)).sin = x
        (arccos(x)).cos = x
    }
}

/// Negation maps the interval [l, u] into [-u, -l].
theorem neg_in_interval(x: Real, l: Real, u: Real) {
    l <= x and x <= u implies -u <= -x and -x <= -l
} by {
    if l <= x and x <= u {
        if not -x <= -l {
            not_lte_imp_gt(-x, -l)
            -x > -l
            -l < -x
            neg_lt_neg_swap_neg(l, x)
            x < l
            false
        }
        -x <= -l
        if not -u <= -x {
            not_lte_imp_gt(-u, -x)
            -u > -x
            -x < -u
            neg_lt_neg_swap_neg(x, u)
            u < x
            false
        }
        -u <= -x
        -u <= -x and -x <= -l
    }
}

/// Cosine is nonnegative on the closed interval [-pi/2, pi/2].
theorem cos_nonneg_on_closed_interval(x: Real) {
    -pi_over_two <= x and x <= pi_over_two implies Real.0 <= x.cos
} by {
    if -pi_over_two <= x and x <= pi_over_two {
        if Real.0 <= x {
            cos_nonneg_on_zero_to_pi_over_two(x)
            Real.0 <= x.cos
        } else {
            not Real.0 <= x
            not_lte_imp_gt(Real.0, x)
            x < Real.0
            lt_add_right(x, Real.0, -x)
            x + -x < Real.0 + -x
            x + -x = Real.0
            Real.0 < -x
            lt_imp_lte(Real.0, -x)
            Real.0 <= -x
            neg_in_interval(x, -pi_over_two, pi_over_two)
            -pi_over_two <= -x and -x <= pi_over_two
            -x <= pi_over_two
            cos_nonneg_on_zero_to_pi_over_two(-x)
            Real.0 <= (-x).cos
            cos_neg(x)
            (-x).cos = x.cos
            Real.0 <= x.cos
        }
    }
}

/// Cosine of the arcsine is nonnegative on [-1, 1].
theorem cos_arcsin_nonneg(x: Real) {
    -Real.1 <= x and x <= Real.1 implies Real.0 <= (arcsin(x)).cos
} by {
    if -Real.1 <= x and x <= Real.1 {
        is_arcsin_spec(x)
        is_arcsin(x, arcsin(x))
        is_arcsin(x, arcsin(x)) =
            ((arcsin(x)).sin = x and -pi_over_two <= arcsin(x) and arcsin(x) <= pi_over_two)
        -pi_over_two <= arcsin(x)
        arcsin(x) <= pi_over_two
        cos_nonneg_on_closed_interval(arcsin(x))
        Real.0 <= (arcsin(x)).cos
    }
}

/// A real in [-1, 1] has square at most one.
theorem sq_le_one_of_bounds(x: Real) {
    -Real.1 <= x and x <= Real.1 implies x * x <= Real.1
} by {
    if -Real.1 <= x and x <= Real.1 {
        pow_suc(x, Nat.1)
        x.pow(Nat.1.suc) = x * x.pow(Nat.1)
        Nat.1.suc = Nat.2
        pow_one[Real](x)
        x.pow(Nat.1) = x
        x.pow(Nat.2) = x * x
        if Real.0 <= x {
            sq_lte_base(x)
            x.pow(Nat.2) <= x
            x.pow(Nat.2) = x * x
            x * x <= x
            lte_trans(x * x, x, Real.1)
            x * x <= Real.1
        } else {
            not Real.0 <= x
            not_lte_imp_gt(Real.0, x)
            x < Real.0
            lt_add_right(x, Real.0, -x)
            x + -x < Real.0 + -x
            x + -x = Real.0
            Real.0 < -x
            lt_imp_lte(Real.0, -x)
            Real.0 <= -x
            neg_in_interval(x, -Real.1, Real.1)
            -Real.1 <= -x and -x <= Real.1
            -x <= Real.1
            sq_lte_base(-x)
            (-x).pow(Nat.2) <= -x
            neg_pow_even(x, Nat.1)
            (-x).pow(Nat.2 * Nat.1) = x.pow(Nat.2 * Nat.1)
            Nat.2 * Nat.1 = Nat.2
            (-x).pow(Nat.2) = x.pow(Nat.2)
            x.pow(Nat.2) = x * x
            x * x <= -x
            lte_trans(x * x, -x, Real.1)
            x * x <= Real.1
        }
    }
}

/// The bridge identity: (1 - x^2).sqrt = (arcsin(x)).cos on [-1, 1].
theorem sqrt_one_sub_sq_eq_cos_arcsin(x: Real) {
    -Real.1 <= x and x <= Real.1 implies
    (Real.1 - x * x).sqrt = Option.some((arcsin(x)).cos)
} by {
    if -Real.1 <= x and x <= Real.1 {
        sin_arcsin(x)
        (arcsin(x)).sin = x
        sin_sq_add_cos_sq(arcsin(x))
        (arcsin(x)).sin.pow(Nat.2) + (arcsin(x)).cos.pow(Nat.2) = Real.1
        pow_suc((arcsin(x)).sin, Nat.1)
        (arcsin(x)).sin.pow(Nat.1.suc) = (arcsin(x)).sin * (arcsin(x)).sin.pow(Nat.1)
        Nat.1.suc = Nat.2
        pow_one[Real]((arcsin(x)).sin)
        (arcsin(x)).sin.pow(Nat.1) = (arcsin(x)).sin
        (arcsin(x)).sin.pow(Nat.2) = (arcsin(x)).sin * (arcsin(x)).sin
        pow_suc((arcsin(x)).cos, Nat.1)
        (arcsin(x)).cos.pow(Nat.1.suc) = (arcsin(x)).cos * (arcsin(x)).cos.pow(Nat.1)
        pow_one[Real]((arcsin(x)).cos)
        (arcsin(x)).cos.pow(Nat.1) = (arcsin(x)).cos
        (arcsin(x)).cos.pow(Nat.2) = (arcsin(x)).cos * (arcsin(x)).cos
        (arcsin(x)).sin * (arcsin(x)).sin + (arcsin(x)).cos * (arcsin(x)).cos = Real.1
        (arcsin(x)).sin = x
        x * x + (arcsin(x)).cos * (arcsin(x)).cos = Real.1
        (arcsin(x)).cos * (arcsin(x)).cos = Real.1 - x * x
        sq_le_one_of_bounds(x)
        x * x <= Real.1
        lte_add_right(x * x, Real.1, -x * x)
        x * x + -x * x <= Real.1 + -x * x
        x * x + -x * x = Real.0
        Real.0 <= Real.1 - x * x
        cos_arcsin_nonneg(x)
        Real.0 <= (arcsin(x)).cos
        Real.1 + -x * x = Real.1 - x * x
        x * x + -x * x <= Real.1 + -x * x
        x * x + -x * x = Real.0
        Real.0 <= Real.1 - x * x
        Real.1 - x * x >= Real.0
        (arcsin(x)).cos >= Real.0
        (arcsin(x)).cos * (arcsin(x)).cos = Real.1 - x * x
        sqrt_unique_nonneg(Real.1 - x * x, (arcsin(x)).cos)
        (Real.1 - x * x).sqrt = Option.some((arcsin(x)).cos)
    }
}

/// Sine of the arccosine is cosine of the arcsine on [-1, 1].
theorem sin_arccos(x: Real) {
    -Real.1 <= x and x <= Real.1 implies (arccos(x)).sin = (arcsin(x)).cos
} by {
    if -Real.1 <= x and x <= Real.1 {
        arccos(x) = pi_over_two - arcsin(x)
        sin_pi_over_two_sub(arcsin(x))
        (pi_over_two - arcsin(x)).sin = (arcsin(x)).cos
        (arccos(x)).sin = (arcsin(x)).cos
    }
}

/// The bridge identity: (1 - x^2).sqrt = (arccos(x)).sin on [-1, 1].
theorem sqrt_one_sub_sq_eq_sin_arccos(x: Real) {
    -Real.1 <= x and x <= Real.1 implies
    (Real.1 - x * x).sqrt = Option.some((arccos(x)).sin)
} by {
    if -Real.1 <= x and x <= Real.1 {
        sin_arccos(x)
        (arccos(x)).sin = (arcsin(x)).cos
        sqrt_one_sub_sq_eq_cos_arcsin(x)
        (Real.1 - x * x).sqrt = Option.some((arcsin(x)).cos)
        Option.some((arcsin(x)).cos) = Option.some((arccos(x)).sin)
        (Real.1 - x * x).sqrt = Option.some((arccos(x)).sin)
    }
}

/// The arccosine of a cosine value in [0, pi] is the argument.
theorem arccos_cos(x: Real) {
    Real.0 <= x and x <= pi implies arccos(x.cos) = x
} by {
    if Real.0 <= x and x <= pi {
        arccos(x.cos) = pi_over_two - arcsin(x.cos)
        sin_pi_over_two_sub(x)
        (pi_over_two - x).sin = x.cos
        x.cos = (pi_over_two - x).sin
        if not -x <= Real.0 {
            not_lte_imp_gt(-x, Real.0)
            -x > Real.0
            Real.0 < -x
            lt_neg_swap_neg(Real.0, x)
            x < -Real.0
            neg_zero
            -Real.0 = Real.0
            x < Real.0
            false
        }
        -x <= Real.0
        lte_add_right(-x, Real.0, pi_over_two)
        -x + pi_over_two <= Real.0 + pi_over_two
        -x + pi_over_two = pi_over_two - x
        Real.0 + pi_over_two = pi_over_two
        pi_over_two - x <= pi_over_two
        if not -pi_over_two <= pi_over_two - x {
            not_lte_imp_gt(-pi_over_two, pi_over_two - x)
            -pi_over_two > pi_over_two - x
            pi_over_two - x < -pi_over_two
            lt_add_right(pi_over_two - x, -pi_over_two, x)
            (pi_over_two - x) + x < -pi_over_two + x
            (pi_over_two - x) + x = pi_over_two
            pi_over_two < -pi_over_two + x
            lt_add_right(pi_over_two, -pi_over_two + x, pi_over_two)
            pi_over_two + pi_over_two < (-pi_over_two + x) + pi_over_two
            pi_over_two + pi_over_two = pi
        (-pi_over_two + x) + pi_over_two = x + (-pi_over_two + pi_over_two)
        -pi_over_two + pi_over_two = Real.0
        x + (-pi_over_two + pi_over_two) = x + Real.0
        x + Real.0 = x
        (-pi_over_two + x) + pi_over_two = x
            pi < x
            x <= pi
            false
        }
        -pi_over_two <= pi_over_two - x
        arcsin_sin(pi_over_two - x)
        arcsin((pi_over_two - x).sin) = pi_over_two - x
        arcsin(x.cos) = pi_over_two - x
        arccos(x.cos) = pi_over_two - (pi_over_two - x)
        pi_over_two - (pi_over_two - x) = x
        arccos(x.cos) = x
    }
}

// The arctangent is continuous.
//
// The arctangent is strictly increasing, and around a point y0 = arctan(x0)
// the tangent values tan(y0 - eps0) and tan(y0 + eps0) bracket x0; the inverse
// image of that bracket is a neighborhood of x0 on which arctan stays within
// eps0 of y0.

/// The arctangent is strictly increasing.
theorem arctan_strictly_increasing(x: Real, y: Real) {
    x < y implies arctan(x) < arctan(y)
} by {
    if x < y {
        if not arctan(x) < arctan(y) {
            not_lt_imp_gte(arctan(x), arctan(y))
            arctan(y) <= arctan(x)
            is_arctan_spec(x)
            is_arctan(x, arctan(x))
            is_arctan(x, arctan(x)) =
                (tan(arctan(x)) = x and -pi_over_two < arctan(x) and arctan(x) < pi_over_two)
            -pi_over_two < arctan(x)
            arctan(x) < pi_over_two
            is_arctan_spec(y)
            is_arctan(y, arctan(y))
            is_arctan(y, arctan(y)) =
                (tan(arctan(y)) = y and -pi_over_two < arctan(y) and arctan(y) < pi_over_two)
            -pi_over_two < arctan(y)
            arctan(y) < pi_over_two
            tan_nondecreasing(arctan(y), arctan(x))
            tan(arctan(y)) <= tan(arctan(x))
            tan_arctan(y)
            tan(arctan(y)) = y
            tan_arctan(x)
            tan(arctan(x)) = x
            y <= x
            x < y
            false
        }
        arctan(x) < arctan(y)
    }
}

// The reciprocal arctangent identity.
//
// For positive x, the arctangents of x and of 1/x are complementary: they sum
// to pi/2.  The proof uses the cofunction identity tan(pi/2 - a) = 1/tan(a)
// together with injectivity of the tangent on the principal branch: both
// pi/2 - arctan(x) and arctan(1/x) lie in (0, pi/2) and have tangent 1/x.

/// The tangent of a complementary angle is the reciprocal tangent.
theorem tan_complement_recip(x: Real) {
    x.sin != Real.0 and x.cos != Real.0 implies
    tan(pi_over_two - x) = Real.1 / tan(x)
} by {
    if x.sin != Real.0 and x.cos != Real.0 {
        tan_eq_pointwise_div
        tan = pointwise_div_real(Real.sin, Real.cos)
        pointwise_div_real_apply(Real.sin, Real.cos, pi_over_two - x)
        pointwise_div_real(Real.sin, Real.cos, pi_over_two - x) =
            (pi_over_two - x).sin / (pi_over_two - x).cos
        tan(pi_over_two - x) = (pi_over_two - x).sin / (pi_over_two - x).cos
        sin_pi_over_two_sub(x)
        (pi_over_two - x).sin = x.cos
        cos_pi_over_two_sub(x)
        (pi_over_two - x).cos = x.sin
        tan(pi_over_two - x) = x.cos / x.sin
        inverse_div(x.sin, x.cos)
        (x.sin / x.cos).inverse = x.cos / x.sin
        tan(x) = x.sin / x.cos
        Real.1 / tan(x) = Real.1 / (x.sin / x.cos)
        Real.1 / (x.sin / x.cos) = Real.1 * (x.sin / x.cos).inverse
        Real.1 * (x.sin / x.cos).inverse = (x.sin / x.cos).inverse
        Real.1 / tan(x) = (x.sin / x.cos).inverse
        Real.1 / tan(x) = x.cos / x.sin
        tan(pi_over_two - x) = Real.1 / tan(x)
    }
}

/// The arctangent is positive on positive reals.
theorem arctan_pos_of_pos(x: Real) {
    x > Real.0 implies arctan(x) > Real.0
} by {
    if x > Real.0 {
        arctan_strictly_increasing(Real.0, x)
        arctan(Real.0) < arctan(x)
        arctan_zero
        arctan(Real.0) = Real.0
        Real.0 < arctan(x)
    }
}

/// The arctangent of the reciprocal of a positive number is the complementary angle.
theorem arctan_recip_pos(x: Real) {
    x > Real.0 implies arctan(x) + arctan(Real.1 / x) = pi_over_two
} by {
    if x > Real.0 {
        arctan_pos_of_pos(x)
        arctan(x) > Real.0
        is_arctan_spec(x)
        is_arctan(x, arctan(x))
        is_arctan(x, arctan(x)) =
            (tan(arctan(x)) = x and -pi_over_two < arctan(x) and arctan(x) < pi_over_two)
        -pi_over_two < arctan(x)
        arctan(x) < pi_over_two
        sin_pos_on_pos_lt_pi_over_two(arctan(x))
        (arctan(x)).sin > Real.0
        lt_imp_ne(Real.0, (arctan(x)).sin)
        (arctan(x)).sin != Real.0
        cos_pos_on_open_interval(arctan(x))
        (arctan(x)).cos > Real.0
        lt_imp_ne(Real.0, (arctan(x)).cos)
        (arctan(x)).cos != Real.0
        tan_complement_recip(arctan(x))
        tan(pi_over_two - arctan(x)) = Real.1 / tan(arctan(x))
        tan_arctan(x)
        tan(arctan(x)) = x
        Real.1 / tan(arctan(x)) = Real.1 / x
        tan(pi_over_two - arctan(x)) = Real.1 / x
        tan_arctan(Real.1 / x)
        tan(arctan(Real.1 / x)) = Real.1 / x
        tan(pi_over_two - arctan(x)) = tan(arctan(Real.1 / x))
        is_arctan_spec(Real.1 / x)
        is_arctan(Real.1 / x, arctan(Real.1 / x))
        is_arctan(Real.1 / x, arctan(Real.1 / x)) =
            (tan(arctan(Real.1 / x)) = Real.1 / x and
             -pi_over_two < arctan(Real.1 / x) and arctan(Real.1 / x) < pi_over_two)
        -pi_over_two < arctan(Real.1 / x)
        arctan(Real.1 / x) < pi_over_two
        lt_add_right(arctan(x), pi_over_two, -arctan(x))
        arctan(x) + -arctan(x) < pi_over_two + -arctan(x)
        arctan(x) + -arctan(x) = Real.0
        pi_over_two + -arctan(x) = pi_over_two - arctan(x)
        Real.0 < pi_over_two - arctan(x)
        pi_over_two_pos
        Real.0 < pi_over_two
        lt_neg_lt(Real.0, pi_over_two)
        -pi_over_two < -Real.0
        neg_zero
        -Real.0 = Real.0
        -pi_over_two < Real.0
        lt_trans(-pi_over_two, Real.0, pi_over_two - arctan(x))
        -pi_over_two < pi_over_two - arctan(x)
        arctan(x) > Real.0
        neg_pos_is_neg(arctan(x))
        -arctan(x) < Real.0
        lt_add_right(-arctan(x), Real.0, pi_over_two)
        -arctan(x) + pi_over_two < Real.0 + pi_over_two
        -arctan(x) + pi_over_two = pi_over_two - arctan(x)
        Real.0 + pi_over_two = pi_over_two
        pi_over_two - arctan(x) < pi_over_two
        tan_injective_on_open_interval(pi_over_two - arctan(x), arctan(Real.1 / x))
        pi_over_two - arctan(x) = arctan(Real.1 / x)
        add_comm(arctan(x), pi_over_two - arctan(x))
        arctan(x) + (pi_over_two - arctan(x)) = (pi_over_two - arctan(x)) + arctan(x)
        (pi_over_two - arctan(x)) + arctan(x) = pi_over_two
        arctan(x) + (pi_over_two - arctan(x)) = pi_over_two
        arctan(x) + (pi_over_two - arctan(x)) = arctan(x) + arctan(Real.1 / x)
        arctan(x) + arctan(Real.1 / x) = pi_over_two
    }
}


/// The arctangent is continuous.
theorem arctan_continuous {
    continuous(arctan)
} by {
    forall(x0: Real) {
        forall(eps: Real) {
            if eps.is_positive {
                is_arctan_spec(x0)
                is_arctan(x0, arctan(x0))
                is_arctan(x0, arctan(x0)) =
                    (tan(arctan(x0)) = x0 and -pi_over_two < arctan(x0) and arctan(x0) < pi_over_two)
                -pi_over_two < arctan(x0)
                arctan(x0) < pi_over_two
                lt_imp_minus_pos(arctan(x0), pi_over_two)
                (pi_over_two - arctan(x0)).is_positive
                pos_gt_zero(pi_over_two - arctan(x0))
                pi_over_two - arctan(x0) > Real.0
                lt_imp_minus_pos(-pi_over_two, arctan(x0))
                (arctan(x0) - -pi_over_two).is_positive
                pos_gt_zero(arctan(x0) - -pi_over_two)
                arctan(x0) + pi_over_two > Real.0
                eps_smaller_than_both(eps, arctan(x0) + pi_over_two)
                let eps1: Real satisfy {
                    eps1.is_positive and eps1 < eps and eps1 < arctan(x0) + pi_over_two
                }
                eps1.is_positive
                eps1 < eps
                eps1 < arctan(x0) + pi_over_two
                eps_smaller_than_both(eps1, pi_over_two - arctan(x0))
                let eps0: Real satisfy {
                    eps0.is_positive and eps0 < eps1 and eps0 < pi_over_two - arctan(x0)
                }
                eps0.is_positive
                eps0 < eps1
                eps0 < pi_over_two - arctan(x0)
                lt_trans(eps0, eps1, eps)
                eps0 < eps
                lt_trans(eps0, eps1, arctan(x0) + pi_over_two)
                eps0 < arctan(x0) + pi_over_two
                if not -pi_over_two < arctan(x0) - eps0 {
                    not_lt_imp_gte(-pi_over_two, arctan(x0) - eps0)
                    arctan(x0) - eps0 <= -pi_over_two
                    lte_add_right(arctan(x0) - eps0, -pi_over_two, pi_over_two)
                    (arctan(x0) - eps0) + pi_over_two <= -pi_over_two + pi_over_two
                    -pi_over_two + pi_over_two = Real.0
                    (arctan(x0) - eps0) + pi_over_two <= Real.0
                    (arctan(x0) - eps0) + pi_over_two = arctan(x0) + pi_over_two - eps0
                    arctan(x0) + pi_over_two - eps0 <= Real.0
                    lte_add_right(arctan(x0) + pi_over_two - eps0, Real.0, eps0)
                    arctan(x0) + pi_over_two - eps0 + eps0 <= Real.0 + eps0
                    arctan(x0) + pi_over_two - eps0 + eps0 = arctan(x0) + pi_over_two
                    Real.0 + eps0 = eps0
                    arctan(x0) + pi_over_two <= eps0
                    eps0 < arctan(x0) + pi_over_two
                    false
                }
                -pi_over_two < arctan(x0) - eps0
                lt_add_right(eps0, pi_over_two - arctan(x0), arctan(x0))
                eps0 + arctan(x0) < (pi_over_two - arctan(x0)) + arctan(x0)
                (pi_over_two - arctan(x0)) + arctan(x0) = pi_over_two
                eps0 + arctan(x0) < pi_over_two
                arctan(x0) + eps0 < pi_over_two
                lt_add_pos(arctan(x0), eps0)
                arctan(x0) < arctan(x0) + eps0
                lt_add_pos(arctan(x0) - eps0, eps0)
                arctan(x0) - eps0 < arctan(x0) - eps0 + eps0
                arctan(x0) - eps0 + eps0 = arctan(x0)
                arctan(x0) - eps0 < arctan(x0)
                tan_arctan(x0)
                tan(arctan(x0)) = x0
                tan_strictly_increasing(arctan(x0) - eps0, arctan(x0))
                tan(arctan(x0) - eps0) < tan(arctan(x0))
                tan(arctan(x0) - eps0) < x0
                tan_strictly_increasing(arctan(x0), arctan(x0) + eps0)
                tan(arctan(x0)) < tan(arctan(x0) + eps0)
                x0 < tan(arctan(x0) + eps0)
                lt_imp_minus_pos(tan(arctan(x0) - eps0), x0)
                (x0 - tan(arctan(x0) - eps0)).is_positive
                pos_gt_zero(x0 - tan(arctan(x0) - eps0))
                x0 - tan(arctan(x0) - eps0) > Real.0
                lt_imp_minus_pos(x0, tan(arctan(x0) + eps0))
                (tan(arctan(x0) + eps0) - x0).is_positive
                pos_gt_zero(tan(arctan(x0) + eps0) - x0)
                tan(arctan(x0) + eps0) - x0 > Real.0
                eps_smaller_than_both(x0 - tan(arctan(x0) - eps0),
                    tan(arctan(x0) + eps0) - x0)
                let delta: Real satisfy {
                    delta.is_positive and
                    delta < x0 - tan(arctan(x0) - eps0) and
                    delta < tan(arctan(x0) + eps0) - x0
                }
                delta.is_positive
                delta < x0 - tan(arctan(x0) - eps0)
                delta < tan(arctan(x0) + eps0) - x0
                forall(z: Real) {
                    if z.is_close(x0, delta) {
                        close_imp_bounds(z, x0, delta)
                        z < x0 + delta
                        z > x0 - delta
                        lt_add_right(delta, x0 - tan(arctan(x0) - eps0),
                            tan(arctan(x0) - eps0))
                        delta + tan(arctan(x0) - eps0) < (x0 - tan(arctan(x0) - eps0)) + tan(arctan(x0) - eps0)
                        (x0 - tan(arctan(x0) - eps0)) + tan(arctan(x0) - eps0) = x0
                        delta + tan(arctan(x0) - eps0) < x0
        (x0 - delta) + delta = x0
        tan(arctan(x0) - eps0) + delta < (x0 - delta) + delta
        lt_add_converse(tan(arctan(x0) - eps0), x0 - delta, delta)
                        tan(arctan(x0) - eps0) + delta < x0
                        tan(arctan(x0) - eps0) < x0 - delta
                        lt_trans(tan(arctan(x0) - eps0), x0 - delta, z)
                        tan(arctan(x0) - eps0) < z
                        lt_add_right(delta, tan(arctan(x0) + eps0) - x0, x0)
                        delta + x0 < (tan(arctan(x0) + eps0) - x0) + x0
                        (tan(arctan(x0) + eps0) - x0) + x0 = tan(arctan(x0) + eps0)
                        delta + x0 < tan(arctan(x0) + eps0)
                        x0 + delta < tan(arctan(x0) + eps0)
                        lt_trans(z, x0 + delta, tan(arctan(x0) + eps0))
                        z < tan(arctan(x0) + eps0)
                        is_arctan_spec(z)
                        is_arctan(z, arctan(z))
                        is_arctan(z, arctan(z)) =
                            (tan(arctan(z)) = z and -pi_over_two < arctan(z) and arctan(z) < pi_over_two)
                        -pi_over_two < arctan(z)
                        arctan(z) < pi_over_two
                        if not arctan(z) < arctan(x0) + eps0 {
                            not_lt_imp_gte(arctan(z), arctan(x0) + eps0)
                            arctan(x0) + eps0 <= arctan(z)
        add_lt_lt(-pi_over_two, arctan(x0), Real.0, eps0)
        -pi_over_two + Real.0 < arctan(x0) + eps0
        -pi_over_two + Real.0 = -pi_over_two
        -pi_over_two < arctan(x0) + eps0
        tan_nondecreasing(arctan(x0) + eps0, arctan(z))
                            tan(arctan(x0) + eps0) <= tan(arctan(z))
                            tan_arctan(z)
                            tan(arctan(z)) = z
                            tan(arctan(x0) + eps0) <= z
                            z < tan(arctan(x0) + eps0)
                            false
                        }
                        arctan(z) < arctan(x0) + eps0
                        if not arctan(x0) - eps0 < arctan(z) {
                            not_lt_imp_gte(arctan(x0) - eps0, arctan(z))
                            arctan(z) <= arctan(x0) - eps0
        neg_pos_is_neg(eps0)
        (-eps0).is_negative
        neg_lt_zero(-eps0)
        -eps0 < Real.0
        lt_add_right(-eps0, Real.0, arctan(x0))
        -eps0 + arctan(x0) < Real.0 + arctan(x0)
        -eps0 + arctan(x0) = arctan(x0) - eps0
        Real.0 + arctan(x0) = arctan(x0)
        arctan(x0) - eps0 < arctan(x0)
        lt_trans(arctan(x0) - eps0, arctan(x0), pi_over_two)
        arctan(x0) - eps0 < pi_over_two
        tan_nondecreasing(arctan(z), arctan(x0) - eps0)
                            tan(arctan(z)) <= tan(arctan(x0) - eps0)
                            tan(arctan(z)) = z
                            z <= tan(arctan(x0) - eps0)
                            tan(arctan(x0) - eps0) < z
                            false
                        }
                        arctan(x0) - eps0 < arctan(z)
                        lt_neg_lt(eps0, eps)
                        -eps < -eps0
                        lt_add_right(-eps, -eps0, arctan(x0))
                        -eps + arctan(x0) < -eps0 + arctan(x0)
                        -eps + arctan(x0) = arctan(x0) - eps
                        -eps0 + arctan(x0) = arctan(x0) - eps0
                        arctan(x0) - eps < arctan(x0) - eps0
                        lt_trans(arctan(x0) - eps, arctan(x0) - eps0, arctan(z))
                        arctan(x0) - eps < arctan(z)
                        lt_add_right(eps0, eps, arctan(x0))
                        eps0 + arctan(x0) < eps + arctan(x0)
                        eps0 + arctan(x0) = arctan(x0) + eps0
                        eps + arctan(x0) = arctan(x0) + eps
                        arctan(x0) + eps0 < arctan(x0) + eps
                        lt_trans(arctan(z), arctan(x0) + eps0, arctan(x0) + eps)
                        arctan(z) < arctan(x0) + eps
                        bounds_imp_close(arctan(z), arctan(x0), eps)
                        arctan(z).is_close(arctan(x0), eps)
                    }
                }
                continuous_condition(arctan, x0, delta, eps)
                delta.is_positive and continuous_condition(arctan, x0, delta, eps)
                exists(delta2: Real) {
                    delta2.is_positive and continuous_condition(arctan, x0, delta2, eps)
                }
            }
        }
        continuous_at(arctan, x0) = forall(eps0: Real) {
            eps0.is_positive implies exists(delta: Real) {
                delta.is_positive and continuous_condition(arctan, x0, delta, eps0)
            }
        }
        if not continuous_at(arctan, x0) {
            not forall(eps1: Real) {
                eps1.is_positive implies exists(delta1: Real) {
                    delta1.is_positive and continuous_condition(arctan, x0, delta1, eps1)
                }
            }
            let bad_eps: Real satisfy {
                bad_eps.is_positive and forall(delta1: Real) {
                    not (delta1.is_positive and continuous_condition(arctan, x0, delta1, bad_eps))
                }
            }
            bad_eps.is_positive
            forall(delta1: Real) {
                not (delta1.is_positive and continuous_condition(arctan, x0, delta1, bad_eps))
            }
            forall(eps2: Real) {
                eps2.is_positive implies exists(delta2: Real) {
                    delta2.is_positive and continuous_condition(arctan, x0, delta2, eps2)
                }
            }
            exists(delta3: Real) {
                delta3.is_positive and continuous_condition(arctan, x0, delta3, bad_eps)
            }
            false
        }
        continuous_at(arctan, x0)
    }
    continuous(arctan)
}

// The derivative of the arctangent.
//
// Following `log_has_derivative_at_pos` in `derivative_exp_log.ac`, the
// derivative of the arctangent is obtained from the derivative of the tangent
// by reciprocating the difference quotient: if y = arctan(x), then
//     (arctan(x) - arctan(x0)) / (x - x0) = 1 / ((tan(y) - tan(y0)) / (y - y0)),
// and tan'(y0) = 1/y0.cos^2 = 1/(1 + x0^2).

/// The tangent has derivative 1/x0.cos^2 at every point where x0.cos is nonzero.
theorem tan_has_derivative_at(x0: Real) {
    x0.cos != Real.0 implies
    has_derivative_at(tan, x0, Real.1 / (x0.cos * x0.cos))
} by {
    if x0.cos != Real.0 {
        sin_has_derivative_at(x0)
        has_derivative_at(Real.sin, x0, x0.cos)
        cos_has_derivative_at(x0)
        has_derivative_at(Real.cos, x0, -x0.sin)
        derivative_pointwise_div(Real.sin, Real.cos, x0, x0.cos, -x0.sin)
        has_derivative_at(pointwise_div_real(Real.sin, Real.cos), x0,
            (x0.cos * x0.cos - x0.sin * -x0.sin) / (x0.cos * x0.cos))
        mul_neg_right(x0.sin, x0.sin)
        x0.sin * -x0.sin = -(x0.sin * x0.sin)
        neg_neg(x0.sin * x0.sin)
        -(-(x0.sin * x0.sin)) = x0.sin * x0.sin
        x0.cos * x0.cos - (-(x0.sin * x0.sin)) = x0.cos * x0.cos + x0.sin * x0.sin
        x0.cos * x0.cos - x0.sin * -x0.sin = x0.cos * x0.cos + x0.sin * x0.sin
        pow_suc(x0.sin, Nat.1)
        x0.sin.pow(Nat.1.suc) = x0.sin * x0.sin.pow(Nat.1)
        Nat.1.suc = Nat.2
        pow_one[Real](x0.sin)
        x0.sin.pow(Nat.1) = x0.sin
        x0.sin.pow(Nat.2) = x0.sin * x0.sin
        pow_suc(x0.cos, Nat.1)
        x0.cos.pow(Nat.1.suc) = x0.cos * x0.cos.pow(Nat.1)
        pow_one[Real](x0.cos)
        x0.cos.pow(Nat.1) = x0.cos
        x0.cos.pow(Nat.2) = x0.cos * x0.cos
        sin_sq_add_cos_sq(x0)
        x0.sin.pow(Nat.2) + x0.cos.pow(Nat.2) = Real.1
        x0.cos * x0.cos + x0.sin * x0.sin = Real.1
        x0.cos * x0.cos - x0.sin * -x0.sin = Real.1
        (x0.cos * x0.cos - x0.sin * -x0.sin) / (x0.cos * x0.cos) =
            Real.1 / (x0.cos * x0.cos)
        has_derivative_at(pointwise_div_real(Real.sin, Real.cos), x0, Real.1 / (x0.cos * x0.cos))
        tan_eq_pointwise_div
        tan = pointwise_div_real(Real.sin, Real.cos)
        function_eq_transport_predicate_rev(
            function(h: Real -> Real) {
                has_derivative_at(h, x0, Real.1 / (x0.cos * x0.cos))
            },
            tan,
            pointwise_div_real(Real.sin, Real.cos)
        )
        has_derivative_at(tan, x0, Real.1 / (x0.cos * x0.cos))
    }
}

/// Cosine squared of the arctangent is 1/(1 + x^2).
theorem cos_sq_arctan_eq(x: Real) {
    (arctan(x)).cos * (arctan(x)).cos = Real.1 / (Real.1 + x * x)
} by {
    tan_arctan(x)
    tan(arctan(x)) = x
    is_arctan_spec(x)
    is_arctan(x, arctan(x))
    is_arctan(x, arctan(x)) =
        (tan(arctan(x)) = x and -pi_over_two < arctan(x) and arctan(x) < pi_over_two)
    -pi_over_two < arctan(x)
    arctan(x) < pi_over_two
    cos_pos_on_open_interval(arctan(x))
    (arctan(x)).cos > Real.0
    lt_imp_ne(Real.0, (arctan(x)).cos)
    (arctan(x)).cos != Real.0
    tan_mul_cos(arctan(x))
    tan(arctan(x)) * (arctan(x)).cos = (arctan(x)).sin
    tan(arctan(x)) = x
    x * (arctan(x)).cos = (arctan(x)).sin
    sin_sq_add_cos_sq(arctan(x))
    (arctan(x)).sin.pow(Nat.2) + (arctan(x)).cos.pow(Nat.2) = Real.1
    pow_suc((arctan(x)).sin, Nat.1)
    (arctan(x)).sin.pow(Nat.1.suc) = (arctan(x)).sin * (arctan(x)).sin.pow(Nat.1)
    Nat.1.suc = Nat.2
    pow_one[Real]((arctan(x)).sin)
    (arctan(x)).sin.pow(Nat.1) = (arctan(x)).sin
    (arctan(x)).sin.pow(Nat.2) = (arctan(x)).sin * (arctan(x)).sin
    pow_suc((arctan(x)).cos, Nat.1)
    (arctan(x)).cos.pow(Nat.1.suc) = (arctan(x)).cos * (arctan(x)).cos.pow(Nat.1)
    pow_one[Real]((arctan(x)).cos)
    (arctan(x)).cos.pow(Nat.1) = (arctan(x)).cos
    (arctan(x)).cos.pow(Nat.2) = (arctan(x)).cos * (arctan(x)).cos
    (arctan(x)).sin * (arctan(x)).sin + (arctan(x)).cos * (arctan(x)).cos = Real.1
    (arctan(x)).sin = x * (arctan(x)).cos
    (x * (arctan(x)).cos) * (x * (arctan(x)).cos) + (arctan(x)).cos * (arctan(x)).cos = Real.1
    (x * (arctan(x)).cos) * (x * (arctan(x)).cos) = (x * x) * ((arctan(x)).cos * (arctan(x)).cos)
    (x * x) * ((arctan(x)).cos * (arctan(x)).cos) + (arctan(x)).cos * (arctan(x)).cos = Real.1
    (Real.1 + x * x) * ((arctan(x)).cos * (arctan(x)).cos) = Real.1
        square_nonneg(x)
        x * x >= Real.0
        Real.0 <= x * x
        Real.0 <= x.pow(Nat.2)
    x.pow(Nat.2) = x * x
        Real.0 <= x * x
        Real.0 < Real.1
        lt_imp_lte(Real.0, Real.1)
        Real.0 <= Real.1
        lte_add_right(Real.0, x * x, Real.1)
        Real.0 + Real.1 <= x * x + Real.1
        Real.0 + Real.1 = Real.1
        x * x + Real.1 = Real.1 + x * x
        Real.1 <= Real.1 + x * x
    Real.0 + x * x = x * x
        two_positive
        two > Real.0
        Real.1 + Real.1 = two
        Real.1 + Real.1 > Real.0
        lt_of_lt_of_lte(Real.0, Real.1, Real.1 + x * x)
        Real.1 + x * x > Real.0
    lt_imp_ne(Real.0, Real.1 + x * x)
    Real.1 + x * x != Real.0
    prod_eq_to_div_eq((arctan(x)).cos * (arctan(x)).cos, Real.1 + x * x, Real.1, Real.1)
    ((arctan(x)).cos * (arctan(x)).cos) / Real.1 = Real.1 / (Real.1 + x * x)
    div_mul_cancel_left(Real.1, (arctan(x)).cos * (arctan(x)).cos)
    (Real.1 * ((arctan(x)).cos * (arctan(x)).cos)) / Real.1 = (arctan(x)).cos * (arctan(x)).cos
    Real.1 * ((arctan(x)).cos * (arctan(x)).cos) = (arctan(x)).cos * (arctan(x)).cos
    (arctan(x)).cos * (arctan(x)).cos = Real.1 / (Real.1 + x * x)
}

/// The arctangent is locally close to its value near any point.
theorem arctan_close(x0: Real, delta2: Real) {
    delta2.is_positive implies exists(delta3: Real) {
        delta3.is_positive and forall(x: Real) {
            x.is_close(x0, delta3) implies arctan(x).is_close(arctan(x0), delta2)
        }
    }
} by {
    if delta2.is_positive {
        arctan_continuous
        continuous(arctan)
        continuous_imp_continuous_at(arctan, x0)
        continuous_at(arctan, x0)
        continuous_at(arctan, x0) = forall(eps0: Real) {
            eps0.is_positive implies exists(delta0: Real) {
                delta0.is_positive and continuous_condition(arctan, x0, delta0, eps0)
            }
        }
        delta2.is_positive
        exists(delta0: Real) {
            delta0.is_positive and continuous_condition(arctan, x0, delta0, delta2)
        }
        let delta3: Real satisfy {
            delta3.is_positive and continuous_condition(arctan, x0, delta3, delta2)
        }
        delta3.is_positive
        continuous_condition(arctan, x0, delta3, delta2)
        continuous_condition(arctan, x0, delta3, delta2) = forall(x1: Real) {
            x1.is_close(x0, delta3) implies arctan(x1).is_close(arctan(x0), delta2)
        }
        forall(x: Real) {
            if x.is_close(x0, delta3) {
                arctan(x).is_close(arctan(x0), delta2)
            }
        }
        delta3.is_positive and forall(x: Real) {
            x.is_close(x0, delta3) implies arctan(x).is_close(arctan(x0), delta2)
        }
        exists(delta4: Real) {
            delta4.is_positive and forall(x: Real) {
                x.is_close(x0, delta4) implies arctan(x).is_close(arctan(x0), delta2)
            }
        }
    }
}

/// The arctangent has derivative 1/(1 + x0^2) at every point x0.
theorem arctan_has_derivative_at(x0: Real) {
    has_derivative_at(arctan, x0, Real.1 / (Real.1 + x0 * x0))
} by {
    is_arctan_spec(x0)
    is_arctan(x0, arctan(x0))
    is_arctan(x0, arctan(x0)) =
        (tan(arctan(x0)) = x0 and -pi_over_two < arctan(x0) and arctan(x0) < pi_over_two)
    -pi_over_two < arctan(x0)
    arctan(x0) < pi_over_two
    cos_pos_on_open_interval(arctan(x0))
    (arctan(x0)).cos > Real.0
    Real.0 < (arctan(x0)).cos
    lt_imp_ne(Real.0, (arctan(x0)).cos)
    (arctan(x0)).cos != Real.0
    mul_pos_pos((arctan(x0)).cos, (arctan(x0)).cos)
    Real.0 < (arctan(x0)).cos * (arctan(x0)).cos
    inverse_of_positive_is_positive((arctan(x0)).cos * (arctan(x0)).cos)
    Real.0 < ((arctan(x0)).cos * (arctan(x0)).cos).inverse
    Real.1 / ((arctan(x0)).cos * (arctan(x0)).cos) =
        Real.1 * ((arctan(x0)).cos * (arctan(x0)).cos).inverse
    Real.1 * ((arctan(x0)).cos * (arctan(x0)).cos).inverse =
        ((arctan(x0)).cos * (arctan(x0)).cos).inverse
    Real.1 / ((arctan(x0)).cos * (arctan(x0)).cos) =
        ((arctan(x0)).cos * (arctan(x0)).cos).inverse
    Real.0 < Real.1 / ((arctan(x0)).cos * (arctan(x0)).cos)
    lt_imp_ne(Real.0, Real.1 / ((arctan(x0)).cos * (arctan(x0)).cos))
    Real.1 / ((arctan(x0)).cos * (arctan(x0)).cos) != Real.0
    tan_has_derivative_at(arctan(x0))
    has_derivative_at(tan, arctan(x0), Real.1 / ((arctan(x0)).cos * (arctan(x0)).cos))
    forall(eps: Real) {
        if eps.is_positive {
            continuous_at_reciprocal_real(Real.1 / ((arctan(x0)).cos * (arctan(x0)).cos))
            continuous_at(reciprocal_real, Real.1 / ((arctan(x0)).cos * (arctan(x0)).cos))
            continuous_at(reciprocal_real, Real.1 / ((arctan(x0)).cos * (arctan(x0)).cos)) =
                forall(eps2: Real) {
                    eps2.is_positive implies exists(delta2: Real) {
                        delta2.is_positive and
                        continuous_condition(reciprocal_real,
                            Real.1 / ((arctan(x0)).cos * (arctan(x0)).cos), delta2, eps2)
                    }
                }
            exists(delta_r: Real) {
                delta_r.is_positive and
                continuous_condition(reciprocal_real,
                    Real.1 / ((arctan(x0)).cos * (arctan(x0)).cos), delta_r, eps)
            }
            let delta_r: Real satisfy {
                delta_r.is_positive and
                continuous_condition(reciprocal_real,
                    Real.1 / ((arctan(x0)).cos * (arctan(x0)).cos), delta_r, eps)
            }
            delta_r.is_positive
            has_derivative_at(tan, arctan(x0), Real.1 / ((arctan(x0)).cos * (arctan(x0)).cos)) =
                forall(eps3: Real) {
                    eps3.is_positive implies exists(delta3: Real) {
                        delta3.is_positive and forall(y: Real) {
                            y != arctan(x0) and y.is_close(arctan(x0), delta3)
                            implies difference_quotient(tan, arctan(x0), y).is_close(
                                Real.1 / ((arctan(x0)).cos * (arctan(x0)).cos), eps3)
                        }
                    }
                }
            delta_r.is_positive
            exists(delta2: Real) {
                delta2.is_positive and forall(y: Real) {
                    y != arctan(x0) and y.is_close(arctan(x0), delta2)
                    implies difference_quotient(tan, arctan(x0), y).is_close(
                        Real.1 / ((arctan(x0)).cos * (arctan(x0)).cos), delta_r)
                }
            }
            let delta2: Real satisfy {
                delta2.is_positive and forall(y: Real) {
                    y != arctan(x0) and y.is_close(arctan(x0), delta2)
                    implies difference_quotient(tan, arctan(x0), y).is_close(
                        Real.1 / ((arctan(x0)).cos * (arctan(x0)).cos), delta_r)
                }
            }
            delta2.is_positive
            arctan_close(x0, delta2)
            let delta3: Real satisfy {
                delta3.is_positive and forall(x: Real) {
                    x.is_close(x0, delta3) implies arctan(x).is_close(arctan(x0), delta2)
                }
            }
            delta3.is_positive
            forall(x: Real) {
                if x != x0 and x.is_close(x0, delta3) {
                    arctan(x).is_close(arctan(x0), delta2)
                    if arctan(x) = arctan(x0) {
                        tan_arctan(x)
                        tan(arctan(x)) = x
                        tan_arctan(x0)
                        tan(arctan(x0)) = x0
                        arctan(x) = arctan(x0)
                        tan(arctan(x)) = tan(arctan(x0))
                        x = x0
                        false
                    }
                    arctan(x) != arctan(x0)
                    difference_quotient(tan, arctan(x0), arctan(x)).is_close(
                        Real.1 / ((arctan(x0)).cos * (arctan(x0)).cos), delta_r)
                    continuous_condition(reciprocal_real,
                        Real.1 / ((arctan(x0)).cos * (arctan(x0)).cos), delta_r, eps) =
                        forall(x1: Real) {
                            x1.is_close(Real.1 / ((arctan(x0)).cos * (arctan(x0)).cos), delta_r)
                            implies reciprocal_real(x1).is_close(
                                reciprocal_real(Real.1 / ((arctan(x0)).cos * (arctan(x0)).cos)), eps)
                        }
                    reciprocal_real(difference_quotient(tan, arctan(x0), arctan(x))).is_close(
                        reciprocal_real(Real.1 / ((arctan(x0)).cos * (arctan(x0)).cos)), eps)
                    difference_quotient(arctan, x0, x) = (arctan(x) - arctan(x0)) / (x - x0)
                    difference_quotient(tan, arctan(x0), arctan(x)) =
                        (tan(arctan(x)) - tan(arctan(x0))) / (arctan(x) - arctan(x0))
                    tan_arctan(x)
                    tan(arctan(x)) = x
                    tan_arctan(x0)
                    tan(arctan(x0)) = x0
                    difference_quotient(tan, arctan(x0), arctan(x)) =
                        (x - x0) / (arctan(x) - arctan(x0))
                    sub_ne_zero_of_ne(x, x0)
                    x - x0 != Real.0
                    sub_ne_zero_of_ne(arctan(x), arctan(x0))
                    arctan(x) - arctan(x0) != Real.0
                    inverse_div(x - x0, arctan(x) - arctan(x0))
                    ((x - x0) / (arctan(x) - arctan(x0))).inverse =
                        (arctan(x) - arctan(x0)) / (x - x0)
                    (difference_quotient(tan, arctan(x0), arctan(x))).inverse =
                        (arctan(x) - arctan(x0)) / (x - x0)
                    reciprocal_real(difference_quotient(tan, arctan(x0), arctan(x))) =
                        (difference_quotient(tan, arctan(x0), arctan(x))).inverse
                    reciprocal_real(difference_quotient(tan, arctan(x0), arctan(x))) =
                        difference_quotient(arctan, x0, x)
                    difference_quotient(arctan, x0, x).is_close(
                        reciprocal_real(Real.1 / ((arctan(x0)).cos * (arctan(x0)).cos)), eps)
                    reciprocal_real_apply(Real.1 / ((arctan(x0)).cos * (arctan(x0)).cos))
                    reciprocal_real(Real.1 / ((arctan(x0)).cos * (arctan(x0)).cos)) =
                        (Real.1 / ((arctan(x0)).cos * (arctan(x0)).cos)).inverse
                    inverse_div(Real.1, (arctan(x0)).cos * (arctan(x0)).cos)
                    (Real.1 / ((arctan(x0)).cos * (arctan(x0)).cos)).inverse =
                        ((arctan(x0)).cos * (arctan(x0)).cos) / Real.1
                    div_mul_cancel_left(Real.1, (arctan(x0)).cos * (arctan(x0)).cos)
                    (Real.1 * ((arctan(x0)).cos * (arctan(x0)).cos)) / Real.1 =
                        (arctan(x0)).cos * (arctan(x0)).cos
                    Real.1 * ((arctan(x0)).cos * (arctan(x0)).cos) =
                        (arctan(x0)).cos * (arctan(x0)).cos
                    ((arctan(x0)).cos * (arctan(x0)).cos) / Real.1 =
                        (arctan(x0)).cos * (arctan(x0)).cos
                    reciprocal_real(Real.1 / ((arctan(x0)).cos * (arctan(x0)).cos)) =
                        (arctan(x0)).cos * (arctan(x0)).cos
                    cos_sq_arctan_eq(x0)
                    (arctan(x0)).cos * (arctan(x0)).cos = Real.1 / (Real.1 + x0 * x0)
                    reciprocal_real(Real.1 / ((arctan(x0)).cos * (arctan(x0)).cos)) =
                        Real.1 / (Real.1 + x0 * x0)
                    difference_quotient(arctan, x0, x).is_close(
                        Real.1 / (Real.1 + x0 * x0), eps)
                }
            }
            exists(delta4: Real) {
                delta4.is_positive and forall(x: Real) {
                    x != x0 and x.is_close(x0, delta4)
                    implies difference_quotient(arctan, x0, x).is_close(
                        Real.1 / (Real.1 + x0 * x0), eps)
                }
            }
        }
    }
    has_derivative_at(arctan, x0, Real.1 / (Real.1 + x0 * x0)) = forall(eps: Real) {
        eps.is_positive implies exists(delta: Real) {
            delta.is_positive and forall(x: Real) {
                x != x0 and x.is_close(x0, delta)
                implies difference_quotient(arctan, x0, x).is_close(
                    Real.1 / (Real.1 + x0 * x0), eps)
            }
        }
    }
    if not has_derivative_at(arctan, x0, Real.1 / (Real.1 + x0 * x0)) {
        not forall(eps: Real) {
            eps.is_positive implies exists(delta: Real) {
                delta.is_positive and forall(x: Real) {
                    x != x0 and x.is_close(x0, delta)
                    implies difference_quotient(arctan, x0, x).is_close(
                        Real.1 / (Real.1 + x0 * x0), eps)
                }
            }
        }
        let bad_eps: Real satisfy {
            bad_eps.is_positive and forall(delta: Real) {
                not (delta.is_positive and forall(x: Real) {
                    x != x0 and x.is_close(x0, delta)
                    implies difference_quotient(arctan, x0, x).is_close(
                        Real.1 / (Real.1 + x0 * x0), bad_eps)
                })
            }
        }
        exists(delta: Real) {
            delta.is_positive and forall(x: Real) {
                x != x0 and x.is_close(x0, delta)
                implies difference_quotient(arctan, x0, x).is_close(
                    Real.1 / (Real.1 + x0 * x0), bad_eps)
            }
        }
        false
    }
    has_derivative_at(arctan, x0, Real.1 / (Real.1 + x0 * x0))
}

/// The arctangent is differentiable with reciprocal-quadratic derivative.
theorem arctan_is_derivative_fn {
    is_derivative_fn(arctan, function(x: Real) { Real.1 / (Real.1 + x * x) })
} by {
    forall(x: Real) {
        arctan_has_derivative_at(x)
        has_derivative_at(arctan, x, Real.1 / (Real.1 + x * x))
        function(x0: Real) { Real.1 / (Real.1 + x0 * x0) }(x) = Real.1 / (Real.1 + x * x)
        has_derivative_at(arctan, x, function(x0: Real) { Real.1 / (Real.1 + x0 * x0) }(x))
    }
    is_derivative_fn(arctan, function(x: Real) { Real.1 / (Real.1 + x * x) }) =
        forall(y: Real) {
            has_derivative_at(arctan, y, function(x: Real) { Real.1 / (Real.1 + x * x) }(y))
        }
    is_derivative_fn(arctan, function(x: Real) { Real.1 / (Real.1 + x * x) })
}

// The arctangent addition formula.

/// Cancelling a common nonzero factor in a quotient of total divisions.
theorem div_cancel_factor(u: Real, v: Real, b: Real) {
    b != Real.0 implies (b * u) / (b * v) = u / v
} by {
    if b != Real.0 {
        if v != Real.0 {
            div_cancel_common(u, b, v)
            (u * b) / (v * b) = u / v
            b * u = u * b
            b * v = v * b
            (b * u) / (b * v) = (u * b) / (v * b)
            (b * u) / (b * v) = u / v
        } else {
            not v != Real.0
            v = Real.0
            b * v = b * Real.0
            b * Real.0 = Real.0
            (b * u) / (b * v) = (b * u) / Real.0
            (b * u) / Real.0 = (b * u) * Real.0.inverse
            zero_inverse
            Real.0.inverse = Real.0
            (b * u) * Real.0.inverse = (b * u) * Real.0
            mul_zero_right(b * u)
            (b * u) * Real.0 = Real.0
            (b * u) / (b * v) = Real.0
            u / v = u / Real.0
            u / Real.0 = u * Real.0.inverse
            u * Real.0.inverse = u * Real.0
            mul_zero_right(u)
            u * Real.0 = Real.0
            u / v = Real.0
            (b * u) / (b * v) = u / v
        }
    }
}

/// The tangent of a sum of arctangents is the quotient formula.
theorem arctan_add(x: Real, y: Real) {
    tan(arctan(x) + arctan(y)) = (x + y) / (Real.1 - x * y)
} by {
    tan_add_basic(arctan(x), arctan(y))
    tan(arctan(x) + arctan(y)) =
        ((arctan(x)).sin * (arctan(y)).cos + (arctan(x)).cos * (arctan(y)).sin) /
        ((arctan(x)).cos * (arctan(y)).cos - (arctan(x)).sin * (arctan(y)).sin)
    is_arctan_spec(x)
    is_arctan(x, arctan(x))
    is_arctan(x, arctan(x)) =
        (tan(arctan(x)) = x and -pi_over_two < arctan(x) and arctan(x) < pi_over_two)
    -pi_over_two < arctan(x)
    arctan(x) < pi_over_two
    cos_pos_on_open_interval(arctan(x))
    (arctan(x)).cos > Real.0
    lt_imp_ne(Real.0, (arctan(x)).cos)
    (arctan(x)).cos != Real.0
    is_arctan_spec(y)
    is_arctan(y, arctan(y))
    is_arctan(y, arctan(y)) =
        (tan(arctan(y)) = y and -pi_over_two < arctan(y) and arctan(y) < pi_over_two)
    -pi_over_two < arctan(y)
    arctan(y) < pi_over_two
    cos_pos_on_open_interval(arctan(y))
    (arctan(y)).cos > Real.0
    lt_imp_ne(Real.0, (arctan(y)).cos)
    (arctan(y)).cos != Real.0
    tan_mul_cos(arctan(x))
    tan(arctan(x)) * (arctan(x)).cos = (arctan(x)).sin
    tan_arctan(x)
    tan(arctan(x)) = x
    x * (arctan(x)).cos = (arctan(x)).sin
    tan_mul_cos(arctan(y))
    tan(arctan(y)) * (arctan(y)).cos = (arctan(y)).sin
    tan_arctan(y)
    tan(arctan(y)) = y
    y * (arctan(y)).cos = (arctan(y)).sin
    (arctan(x)).sin * (arctan(y)).cos + (arctan(x)).cos * (arctan(y)).sin =
        (x * (arctan(x)).cos) * (arctan(y)).cos + (arctan(x)).cos * (y * (arctan(y)).cos)
    (x * (arctan(x)).cos) * (arctan(y)).cos + (arctan(x)).cos * (y * (arctan(y)).cos) =
        (x + y) * ((arctan(x)).cos * (arctan(y)).cos)
    (x + y) * ((arctan(x)).cos * (arctan(y)).cos) =
        ((arctan(x)).cos * (arctan(y)).cos) * (x + y)
    (arctan(x)).sin * (arctan(y)).cos + (arctan(x)).cos * (arctan(y)).sin =
        ((arctan(x)).cos * (arctan(y)).cos) * (x + y)
    (arctan(x)).sin * (arctan(y)).sin = (x * (arctan(x)).cos) * (y * (arctan(y)).cos)
    (x * (arctan(x)).cos) * (y * (arctan(y)).cos) = (x * y) * ((arctan(x)).cos * (arctan(y)).cos)
    (arctan(x)).cos * (arctan(y)).cos - (arctan(x)).sin * (arctan(y)).sin =
        (arctan(x)).cos * (arctan(y)).cos - (x * y) * ((arctan(x)).cos * (arctan(y)).cos)
    (arctan(x)).cos * (arctan(y)).cos - (x * y) * ((arctan(x)).cos * (arctan(y)).cos) =
        (Real.1 - x * y) * ((arctan(x)).cos * (arctan(y)).cos)
    (Real.1 - x * y) * ((arctan(x)).cos * (arctan(y)).cos) =
        ((arctan(x)).cos * (arctan(y)).cos) * (Real.1 - x * y)
    (arctan(x)).cos * (arctan(y)).cos - (arctan(x)).sin * (arctan(y)).sin =
        ((arctan(x)).cos * (arctan(y)).cos) * (Real.1 - x * y)
    tan(arctan(x) + arctan(y)) =
        (((arctan(x)).cos * (arctan(y)).cos) * (x + y)) /
        (((arctan(x)).cos * (arctan(y)).cos) * (Real.1 - x * y))
    mul_not_zero((arctan(x)).cos, (arctan(y)).cos)
    (arctan(x)).cos * (arctan(y)).cos != Real.0
    div_cancel_factor(x + y, Real.1 - x * y, (arctan(x)).cos * (arctan(y)).cos)
    (((arctan(x)).cos * (arctan(y)).cos) * (x + y)) /
        (((arctan(x)).cos * (arctan(y)).cos) * (Real.1 - x * y)) = (x + y) / (Real.1 - x * y)
    tan(arctan(x) + arctan(y)) = (x + y) / (Real.1 - x * y)
}

// The derivative of the arcsine.
//
// The arcsine is continuous on the open interval (-1, 1) by the same bracket
// argument used for the arctangent, and its derivative is the reciprocal of
// (arcsin x).cos, obtained from the derivative of sine exactly as the
// arctangent derivative was obtained from the derivative of tangent.

/// Sine is nondecreasing on [-pi/2, pi/2].
theorem sin_nondecreasing(x: Real, y: Real) {
    -pi_over_two <= x and x <= y and y <= pi_over_two implies x.sin <= y.sin
} by {
    if -pi_over_two <= x and x <= y and y <= pi_over_two {
        if x < y {
            sin_strictly_increasing(x, y)
            x.sin < y.sin
            lt_imp_lte(x.sin, y.sin)
            x.sin <= y.sin
        } else {
            not x < y
            not_lt_imp_gte(x, y)
            y <= x
            lte_antisymm(x, y)
            x = y
            y = x
            y.sin = x.sin
            lte_refl(x.sin)
            x.sin <= x.sin
            x.sin <= y.sin
        }
    }
}

/// Sine lies between negative one and one on [-pi/2, pi/2].
theorem sin_between_neg_one_one(t: Real) {
    -pi_over_two <= t and t <= pi_over_two implies
    -Real.1 <= t.sin and t.sin <= Real.1
} by {
    if -pi_over_two <= t and t <= pi_over_two {
        sin_neg(pi_over_two)
        (-pi_over_two).sin = -pi_over_two.sin
        sin_pi_over_two_one
        pi_over_two.sin = Real.1
        (-pi_over_two).sin = -Real.1
        sin_nondecreasing(-pi_over_two, t)
        (-pi_over_two).sin <= t.sin
        -Real.1 <= t.sin
        sin_nondecreasing(t, pi_over_two)
        t.sin <= pi_over_two.sin
        t.sin <= Real.1
        -Real.1 <= t.sin and t.sin <= Real.1
    }
}

/// The arcsine of a point of (-1, 1) lies strictly inside (-pi/2, pi/2).
theorem arcsin_in_open_interval(x: Real) {
    -Real.1 < x and x < Real.1 implies
    -pi_over_two < arcsin(x) and arcsin(x) < pi_over_two
} by {
    if -Real.1 < x and x < Real.1 {
        lt_imp_lte(-Real.1, x)
        -Real.1 <= x
        lt_imp_lte(x, Real.1)
        x <= Real.1
        is_arcsin_spec(x)
        is_arcsin(x, arcsin(x))
        is_arcsin(x, arcsin(x)) =
            ((arcsin(x)).sin = x and -pi_over_two <= arcsin(x) and arcsin(x) <= pi_over_two)
        -pi_over_two <= arcsin(x)
        arcsin(x) <= pi_over_two
        if not arcsin(x) < pi_over_two {
            not_lt_imp_gte(arcsin(x), pi_over_two)
            arcsin(x) >= pi_over_two
            pi_over_two <= arcsin(x)
            lte_antisymm(arcsin(x), pi_over_two)
            arcsin(x) = pi_over_two
            sin_arcsin(x)
            (arcsin(x)).sin = x
            (arcsin(x)).sin = pi_over_two.sin
            sin_pi_over_two_one
            pi_over_two.sin = Real.1
            x = Real.1
            x < Real.1
            false
        }
        arcsin(x) < pi_over_two
        if not -pi_over_two < arcsin(x) {
            not_lt_imp_gte(-pi_over_two, arcsin(x))
            arcsin(x) <= -pi_over_two
            lte_antisymm(-pi_over_two, arcsin(x))
            arcsin(x) = -pi_over_two
            sin_arcsin(x)
            (arcsin(x)).sin = x
            (arcsin(x)).sin = (-pi_over_two).sin
            sin_neg(pi_over_two)
            (-pi_over_two).sin = -pi_over_two.sin
            sin_pi_over_two_one
            pi_over_two.sin = Real.1
            (-pi_over_two).sin = -Real.1
            x = -Real.1
            -Real.1 < x
            false
        }
        -pi_over_two < arcsin(x)
        -pi_over_two < arcsin(x) and arcsin(x) < pi_over_two
    }
}

/// The arcsine is strictly increasing on [-1, 1].
theorem arcsin_strictly_increasing(x: Real, y: Real) {
    -Real.1 <= x and x < y and y <= Real.1 implies arcsin(x) < arcsin(y)
} by {
    if -Real.1 <= x and x < y and y <= Real.1 {
        if not arcsin(x) < arcsin(y) {
            not_lt_imp_gte(arcsin(x), arcsin(y))
            arcsin(y) <= arcsin(x)
            lt_of_lt_of_lte(x, y, Real.1)
            x < Real.1
            lt_imp_lte(x, Real.1)
            x <= Real.1
            is_arcsin_spec(x)
            is_arcsin(x, arcsin(x))
            is_arcsin(x, arcsin(x)) =
                ((arcsin(x)).sin = x and -pi_over_two <= arcsin(x) and arcsin(x) <= pi_over_two)
            -pi_over_two <= arcsin(x)
            arcsin(x) <= pi_over_two
            lt_of_lte_of_lt(-Real.1, x, y)
            -Real.1 < y
            lt_imp_lte(-Real.1, y)
            -Real.1 <= y
            lt_imp_lte(y, Real.1)
            y <= Real.1
            is_arcsin_spec(y)
            is_arcsin(y, arcsin(y))
            is_arcsin(y, arcsin(y)) =
                ((arcsin(y)).sin = y and -pi_over_two <= arcsin(y) and arcsin(y) <= pi_over_two)
            -pi_over_two <= arcsin(y)
            arcsin(y) <= pi_over_two
            sin_nondecreasing(arcsin(y), arcsin(x))
            (arcsin(y)).sin <= (arcsin(x)).sin
            sin_arcsin(y)
            (arcsin(y)).sin = y
            sin_arcsin(x)
            (arcsin(x)).sin = x
            y <= x
            x < y
            false
        }
        arcsin(x) < arcsin(y)
        arcsin(x) < arcsin(y)
    }
}

/// The arcsine is continuous at every point of (-1, 1).
theorem arcsin_continuous_at(x0: Real) {
    -Real.1 < x0 and x0 < Real.1 implies continuous_at(arcsin, x0)
} by {
    if -Real.1 < x0 and x0 < Real.1 {
        arcsin_in_open_interval(x0)
        -pi_over_two < arcsin(x0) and arcsin(x0) < pi_over_two
        -pi_over_two < arcsin(x0)
        arcsin(x0) < pi_over_two
        lt_imp_lte(-Real.1, x0)
        -Real.1 <= x0
        lt_imp_lte(x0, Real.1)
        x0 <= Real.1
        sin_arcsin(x0)
        (arcsin(x0)).sin = x0
        forall(eps: Real) {
            if eps.is_positive {
                lt_imp_minus_pos(arcsin(x0), pi_over_two)
                (pi_over_two - arcsin(x0)).is_positive
                pos_gt_zero(pi_over_two - arcsin(x0))
                pi_over_two - arcsin(x0) > Real.0
                lt_imp_minus_pos(-pi_over_two, arcsin(x0))
                (arcsin(x0) - -pi_over_two).is_positive
                pos_gt_zero(arcsin(x0) - -pi_over_two)
                arcsin(x0) + pi_over_two > Real.0
                eps_smaller_than_both(eps, arcsin(x0) + pi_over_two)
                let eps1: Real satisfy {
                    eps1.is_positive and eps1 < eps and eps1 < arcsin(x0) + pi_over_two
                }
                eps1.is_positive
                eps1 < eps
                eps1 < arcsin(x0) + pi_over_two
                eps_smaller_than_both(eps1, pi_over_two - arcsin(x0))
                let eps0: Real satisfy {
                    eps0.is_positive and eps0 < eps1 and eps0 < pi_over_two - arcsin(x0)
                }
                eps0.is_positive
                eps0 < eps1
                eps0 < pi_over_two - arcsin(x0)
                lt_trans(eps0, eps1, eps)
                eps0 < eps
                lt_trans(eps0, eps1, arcsin(x0) + pi_over_two)
                eps0 < arcsin(x0) + pi_over_two
                if not -pi_over_two < arcsin(x0) - eps0 {
                    not_lt_imp_gte(-pi_over_two, arcsin(x0) - eps0)
                    arcsin(x0) - eps0 <= -pi_over_two
                    lte_add_right(arcsin(x0) - eps0, -pi_over_two, pi_over_two)
                    (arcsin(x0) - eps0) + pi_over_two <= -pi_over_two + pi_over_two
                    -pi_over_two + pi_over_two = Real.0
                    (arcsin(x0) - eps0) + pi_over_two <= Real.0
                    (arcsin(x0) - eps0) + pi_over_two = arcsin(x0) + pi_over_two - eps0
                    arcsin(x0) + pi_over_two - eps0 <= Real.0
                    lte_add_right(arcsin(x0) + pi_over_two - eps0, Real.0, eps0)
                    arcsin(x0) + pi_over_two - eps0 + eps0 <= Real.0 + eps0
                    arcsin(x0) + pi_over_two - eps0 + eps0 = arcsin(x0) + pi_over_two
                    Real.0 + eps0 = eps0
                    arcsin(x0) + pi_over_two <= eps0
                    eps0 < arcsin(x0) + pi_over_two
                    false
                }
                -pi_over_two < arcsin(x0) - eps0
                lt_add_right(eps0, pi_over_two - arcsin(x0), arcsin(x0))
                eps0 + arcsin(x0) < (pi_over_two - arcsin(x0)) + arcsin(x0)
                (pi_over_two - arcsin(x0)) + arcsin(x0) = pi_over_two
                eps0 + arcsin(x0) < pi_over_two
                arcsin(x0) + eps0 < pi_over_two
                lt_add_pos(arcsin(x0), eps0)
                arcsin(x0) < arcsin(x0) + eps0
                lt_add_pos(arcsin(x0) - eps0, eps0)
                arcsin(x0) - eps0 < arcsin(x0) - eps0 + eps0
                arcsin(x0) - eps0 + eps0 = arcsin(x0)
                arcsin(x0) - eps0 < arcsin(x0)
                lt_imp_lte(-pi_over_two, arcsin(x0) - eps0)
                -pi_over_two <= arcsin(x0) - eps0
        lt_trans(arcsin(x0) - eps0, arcsin(x0), pi_over_two)
        arcsin(x0) - eps0 < pi_over_two
        lt_imp_lte(arcsin(x0) - eps0, pi_over_two)
        add_lt_lt(-pi_over_two, arcsin(x0), Real.0, eps0)
        -pi_over_two + Real.0 < arcsin(x0) + eps0
        -pi_over_two + Real.0 = -pi_over_two
        -pi_over_two < arcsin(x0) + eps0
        lt_imp_lte(-pi_over_two, arcsin(x0) + eps0)
                lt_imp_lte(arcsin(x0) + eps0, pi_over_two)
                arcsin(x0) + eps0 <= pi_over_two
                sin_strictly_increasing(arcsin(x0) - eps0, arcsin(x0))
                (arcsin(x0) - eps0).sin < (arcsin(x0)).sin
                (arcsin(x0) - eps0).sin < x0
                sin_strictly_increasing(arcsin(x0), arcsin(x0) + eps0)
                (arcsin(x0)).sin < (arcsin(x0) + eps0).sin
                x0 < (arcsin(x0) + eps0).sin
                lt_imp_minus_pos((arcsin(x0) - eps0).sin, x0)
                (x0 - (arcsin(x0) - eps0).sin).is_positive
                pos_gt_zero(x0 - (arcsin(x0) - eps0).sin)
                x0 - (arcsin(x0) - eps0).sin > Real.0
                lt_imp_minus_pos(x0, (arcsin(x0) + eps0).sin)
                ((arcsin(x0) + eps0).sin - x0).is_positive
                pos_gt_zero((arcsin(x0) + eps0).sin - x0)
                (arcsin(x0) + eps0).sin - x0 > Real.0
                eps_smaller_than_both(x0 - (arcsin(x0) - eps0).sin,
                    (arcsin(x0) + eps0).sin - x0)
                let delta: Real satisfy {
                    delta.is_positive and
                    delta < x0 - (arcsin(x0) - eps0).sin and
                    delta < (arcsin(x0) + eps0).sin - x0
                }
                delta.is_positive
                delta < x0 - (arcsin(x0) - eps0).sin
                delta < (arcsin(x0) + eps0).sin - x0
                forall(z: Real) {
                    if z.is_close(x0, delta) {
                        close_imp_bounds(z, x0, delta)
                        z < x0 + delta
                        z > x0 - delta
                        (x0 - delta) + delta = x0
                        lt_add_right(delta, x0 - (arcsin(x0) - eps0).sin,
                            (arcsin(x0) - eps0).sin)
                        delta + (arcsin(x0) - eps0).sin < (x0 - (arcsin(x0) - eps0).sin) + (arcsin(x0) - eps0).sin
                        (x0 - (arcsin(x0) - eps0).sin) + (arcsin(x0) - eps0).sin = x0
                        delta + (arcsin(x0) - eps0).sin < x0
                        (arcsin(x0) - eps0).sin + delta < (x0 - delta) + delta
                        lt_add_converse((arcsin(x0) - eps0).sin, x0 - delta, delta)
                        (arcsin(x0) - eps0).sin < x0 - delta
                        lt_trans((arcsin(x0) - eps0).sin, x0 - delta, z)
                        (arcsin(x0) - eps0).sin < z
                        lt_add_right(delta, (arcsin(x0) + eps0).sin - x0, x0)
                        delta + x0 < ((arcsin(x0) + eps0).sin - x0) + x0
                        ((arcsin(x0) + eps0).sin - x0) + x0 = (arcsin(x0) + eps0).sin
                        delta + x0 < (arcsin(x0) + eps0).sin
                        x0 + delta < (arcsin(x0) + eps0).sin
                        lt_trans(z, x0 + delta, (arcsin(x0) + eps0).sin)
                        z < (arcsin(x0) + eps0).sin
                        sin_between_neg_one_one(arcsin(x0) - eps0)
                        -Real.1 <= (arcsin(x0) - eps0).sin and (arcsin(x0) - eps0).sin <= Real.1
                        -Real.1 <= (arcsin(x0) - eps0).sin
        lt_of_lte_of_lt(-Real.1, (arcsin(x0) - eps0).sin, z)
        -Real.1 < z
        lt_imp_lte(-Real.1, z)
        sin_between_neg_one_one(arcsin(x0) + eps0)
        -Real.1 <= (arcsin(x0) + eps0).sin and (arcsin(x0) + eps0).sin <= Real.1
        (arcsin(x0) + eps0).sin <= Real.1
        lt_of_lt_of_lte(z, (arcsin(x0) + eps0).sin, Real.1)
        z < Real.1
        lt_imp_lte(z, Real.1)
                        is_arcsin_spec(z)
                        is_arcsin(z, arcsin(z))
                        is_arcsin(z, arcsin(z)) =
                            ((arcsin(z)).sin = z and -pi_over_two <= arcsin(z) and arcsin(z) <= pi_over_two)
                        -pi_over_two <= arcsin(z)
                        arcsin(z) <= pi_over_two
                        if not arcsin(z) < arcsin(x0) + eps0 {
                            not_lt_imp_gte(arcsin(z), arcsin(x0) + eps0)
                            arcsin(x0) + eps0 <= arcsin(z)
                            sin_nondecreasing(arcsin(x0) + eps0, arcsin(z))
                            (arcsin(x0) + eps0).sin <= (arcsin(z)).sin
                            sin_arcsin(z)
                            (arcsin(z)).sin = z
                            (arcsin(x0) + eps0).sin <= z
                            z < (arcsin(x0) + eps0).sin
                            false
                        }
                        arcsin(z) < arcsin(x0) + eps0
                        if not arcsin(x0) - eps0 < arcsin(z) {
                            not_lt_imp_gte(arcsin(x0) - eps0, arcsin(z))
                            arcsin(z) <= arcsin(x0) - eps0
                            sin_nondecreasing(arcsin(z), arcsin(x0) - eps0)
                            (arcsin(z)).sin <= (arcsin(x0) - eps0).sin
                            (arcsin(z)).sin = z
                            z <= (arcsin(x0) - eps0).sin
                            (arcsin(x0) - eps0).sin < z
                            false
                        }
                        arcsin(x0) - eps0 < arcsin(z)
                        lt_neg_lt(eps0, eps)
                        -eps < -eps0
                        lt_add_right(-eps, -eps0, arcsin(x0))
                        -eps + arcsin(x0) < -eps0 + arcsin(x0)
                        -eps + arcsin(x0) = arcsin(x0) - eps
                        -eps0 + arcsin(x0) = arcsin(x0) - eps0
                        arcsin(x0) - eps < arcsin(x0) - eps0
                        lt_trans(arcsin(x0) - eps, arcsin(x0) - eps0, arcsin(z))
                        arcsin(x0) - eps < arcsin(z)
                        lt_add_right(eps0, eps, arcsin(x0))
                        eps0 + arcsin(x0) < eps + arcsin(x0)
                        eps0 + arcsin(x0) = arcsin(x0) + eps0
                        eps + arcsin(x0) = arcsin(x0) + eps
                        arcsin(x0) + eps0 < arcsin(x0) + eps
                        lt_trans(arcsin(z), arcsin(x0) + eps0, arcsin(x0) + eps)
                        arcsin(z) < arcsin(x0) + eps
                        bounds_imp_close(arcsin(z), arcsin(x0), eps)
                        arcsin(z).is_close(arcsin(x0), eps)
                    }
                }
                continuous_condition(arcsin, x0, delta, eps)
                delta.is_positive and continuous_condition(arcsin, x0, delta, eps)
                exists(delta2: Real) {
                    delta2.is_positive and continuous_condition(arcsin, x0, delta2, eps)
                }
            }
        }
        continuous_at(arcsin, x0) = forall(eps0: Real) {
            eps0.is_positive implies exists(delta: Real) {
                delta.is_positive and continuous_condition(arcsin, x0, delta, eps0)
            }
        }
        if not continuous_at(arcsin, x0) {
            not forall(eps1: Real) {
                eps1.is_positive implies exists(delta1: Real) {
                    delta1.is_positive and continuous_condition(arcsin, x0, delta1, eps1)
                }
            }
            let bad_eps: Real satisfy {
                bad_eps.is_positive and forall(delta1: Real) {
                    not (delta1.is_positive and continuous_condition(arcsin, x0, delta1, bad_eps))
                }
            }
            bad_eps.is_positive
            forall(eps2: Real) {
                eps2.is_positive implies exists(delta2: Real) {
                    delta2.is_positive and continuous_condition(arcsin, x0, delta2, eps2)
                }
            }
            exists(delta3: Real) {
                delta3.is_positive and continuous_condition(arcsin, x0, delta3, bad_eps)
            }
            false
        }
        continuous_at(arcsin, x0)
    }
}

/// The arcsine is locally close to its value near any point of (-1, 1).
theorem arcsin_close(x0: Real, delta2: Real) {
    -Real.1 < x0 and x0 < Real.1 and delta2.is_positive implies exists(delta3: Real) {
        delta3.is_positive and delta3 < Real.1 - x0 and delta3 < x0 + Real.1 and
        forall(x: Real) {
            x.is_close(x0, delta3) implies arcsin(x).is_close(arcsin(x0), delta2)
        }
    }
} by {
    if -Real.1 < x0 and x0 < Real.1 and delta2.is_positive {
        arcsin_continuous_at(x0)
        continuous_at(arcsin, x0)
        continuous_at(arcsin, x0) = forall(eps0: Real) {
            eps0.is_positive implies exists(delta0: Real) {
                delta0.is_positive and continuous_condition(arcsin, x0, delta0, eps0)
            }
        }
        delta2.is_positive
        exists(delta0: Real) {
            delta0.is_positive and continuous_condition(arcsin, x0, delta0, delta2)
        }
        let delta0: Real satisfy {
            delta0.is_positive and continuous_condition(arcsin, x0, delta0, delta2)
        }
        delta0.is_positive
        continuous_condition(arcsin, x0, delta0, delta2)
        lt_imp_minus_pos(x0, Real.1)
        (Real.1 - x0).is_positive
        pos_gt_zero(Real.1 - x0)
        Real.1 - x0 > Real.0
        lt_imp_minus_pos(-Real.1, x0)
        (x0 - -Real.1).is_positive
        pos_gt_zero(x0 - -Real.1)
        x0 + Real.1 > Real.0
        eps_smaller_than_both(delta0, Real.1 - x0)
        let delta1: Real satisfy {
            delta1.is_positive and delta1 < delta0 and delta1 < Real.1 - x0
        }
        delta1.is_positive
        delta1 < delta0
        delta1 < Real.1 - x0
        eps_smaller_than_both(delta1, x0 + Real.1)
        let delta3: Real satisfy {
            delta3.is_positive and delta3 < delta1 and delta3 < x0 + Real.1
        }
        delta3.is_positive
        delta3 < delta1
        delta3 < x0 + Real.1
        lt_trans(delta3, delta1, delta0)
        delta3 < delta0
        lt_trans(delta3, delta1, Real.1 - x0)
        delta3 < Real.1 - x0
        forall(x: Real) {
            if x.is_close(x0, delta3) {
                x.is_close(x0, delta3) = (x - x0).abs < delta3
                (x - x0).abs < delta3
                lt_trans((x - x0).abs, delta3, delta0)
                (x - x0).abs < delta0
                x.is_close(x0, delta0) = (x - x0).abs < delta0
                x.is_close(x0, delta0)
                continuous_condition(arcsin, x0, delta0, delta2) = forall(x1: Real) {
                    x1.is_close(x0, delta0) implies arcsin(x1).is_close(arcsin(x0), delta2)
                }
                arcsin(x).is_close(arcsin(x0), delta2)
            }
        }
        delta3.is_positive and delta3 < Real.1 - x0 and delta3 < x0 + Real.1 and
        forall(x: Real) {
            x.is_close(x0, delta3) implies arcsin(x).is_close(arcsin(x0), delta2)
        }
        exists(delta4: Real) {
            delta4.is_positive and delta4 < Real.1 - x0 and delta4 < x0 + Real.1 and
            forall(x: Real) {
                x.is_close(x0, delta4) implies arcsin(x).is_close(arcsin(x0), delta2)
            }
        }
    }
}

/// The arcsine has derivative 1/(arcsin(x0)).cos at every point of (-1, 1).
theorem arcsin_has_derivative_at(x0: Real) {
    -Real.1 < x0 and x0 < Real.1 implies
    has_derivative_at(arcsin, x0, Real.1 / (arcsin(x0)).cos)
} by {
    if -Real.1 < x0 and x0 < Real.1 {
        lt_imp_lte(-Real.1, x0)
        -Real.1 <= x0
        lt_imp_lte(x0, Real.1)
        x0 <= Real.1
        arcsin_in_open_interval(x0)
        -pi_over_two < arcsin(x0) and arcsin(x0) < pi_over_two
        -pi_over_two < arcsin(x0)
        arcsin(x0) < pi_over_two
        cos_pos_on_open_interval(arcsin(x0))
        (arcsin(x0)).cos > Real.0
        Real.0 < (arcsin(x0)).cos
        lt_imp_ne(Real.0, (arcsin(x0)).cos)
        (arcsin(x0)).cos != Real.0
        inverse_of_positive_is_positive((arcsin(x0)).cos)
        Real.0 < (arcsin(x0)).cos.inverse
        Real.1 / (arcsin(x0)).cos = Real.1 * (arcsin(x0)).cos.inverse
        Real.1 * (arcsin(x0)).cos.inverse = (arcsin(x0)).cos.inverse
        Real.1 / (arcsin(x0)).cos = (arcsin(x0)).cos.inverse
        Real.0 < Real.1 / (arcsin(x0)).cos
        lt_imp_ne(Real.0, Real.1 / (arcsin(x0)).cos)
        Real.1 / (arcsin(x0)).cos != Real.0
        sin_has_derivative_at(arcsin(x0))
        has_derivative_at(Real.sin, arcsin(x0), (arcsin(x0)).cos)
        forall(eps: Real) {
            if eps.is_positive {
                continuous_at_reciprocal_real((arcsin(x0)).cos)
                continuous_at(reciprocal_real, (arcsin(x0)).cos)
                continuous_at(reciprocal_real, (arcsin(x0)).cos) = forall(eps2: Real) {
                    eps2.is_positive implies exists(delta2: Real) {
                        delta2.is_positive and
                        continuous_condition(reciprocal_real, (arcsin(x0)).cos, delta2, eps2)
                    }
                }
                exists(delta_r: Real) {
                    delta_r.is_positive and
                    continuous_condition(reciprocal_real, (arcsin(x0)).cos, delta_r, eps)
                }
                let delta_r: Real satisfy {
                    delta_r.is_positive and
                    continuous_condition(reciprocal_real, (arcsin(x0)).cos, delta_r, eps)
                }
                delta_r.is_positive
                has_derivative_at(Real.sin, arcsin(x0), (arcsin(x0)).cos) = forall(eps3: Real) {
                    eps3.is_positive implies exists(delta3: Real) {
                        delta3.is_positive and forall(y: Real) {
                            y != arcsin(x0) and y.is_close(arcsin(x0), delta3)
                            implies difference_quotient(Real.sin, arcsin(x0), y).is_close(
                                (arcsin(x0)).cos, eps3)
                        }
                    }
                }
                delta_r.is_positive
                exists(delta2: Real) {
                    delta2.is_positive and forall(y: Real) {
                        y != arcsin(x0) and y.is_close(arcsin(x0), delta2)
                        implies difference_quotient(Real.sin, arcsin(x0), y).is_close(
                            (arcsin(x0)).cos, delta_r)
                    }
                }
                let delta2: Real satisfy {
                    delta2.is_positive and forall(y: Real) {
                        y != arcsin(x0) and y.is_close(arcsin(x0), delta2)
                        implies difference_quotient(Real.sin, arcsin(x0), y).is_close(
                            (arcsin(x0)).cos, delta_r)
                    }
                }
                delta2.is_positive
                arcsin_close(x0, delta2)
                let delta3: Real satisfy {
                    delta3.is_positive and delta3 < Real.1 - x0 and delta3 < x0 + Real.1 and
                    forall(x: Real) {
                        x.is_close(x0, delta3) implies arcsin(x).is_close(arcsin(x0), delta2)
                    }
                }
                delta3.is_positive
                delta3 < Real.1 - x0
                delta3 < x0 + Real.1
                forall(x: Real) {
                    if x != x0 and x.is_close(x0, delta3) {
                        close_imp_bounds(x, x0, delta3)
                        x < x0 + delta3
                        x > x0 - delta3
                        lt_add_right(delta3, Real.1 - x0, x0)
                        delta3 + x0 < (Real.1 - x0) + x0
                        (Real.1 - x0) + x0 = Real.1
                        delta3 + x0 < Real.1
                        x0 + delta3 < Real.1
                        lt_trans(x, x0 + delta3, Real.1)
                        x < Real.1
                        lt_imp_lte(x, Real.1)
                        x <= Real.1
                        lt_add_right(delta3, x0 + Real.1, -Real.1)
                        delta3 + -Real.1 < (x0 + Real.1) + -Real.1
                        (x0 + Real.1) + -Real.1 = x0 + (Real.1 + -Real.1)
                        Real.1 + -Real.1 = Real.0
                        x0 + (Real.1 + -Real.1) = x0 + Real.0
                        x0 + Real.0 = x0
                        (x0 + Real.1) + -Real.1 = x0
                        delta3 + -Real.1 < x0
        delta3 + -Real.1 = -Real.1 + delta3
        -Real.1 + delta3 < x0
        (x0 - delta3) + delta3 = x0
        -Real.1 + delta3 < (x0 - delta3) + delta3
        lt_add_converse(-Real.1, x0 - delta3, delta3)
                        -Real.1 < x0 - delta3
                        lt_trans(-Real.1, x0 - delta3, x)
                        -Real.1 < x
                        lt_imp_lte(-Real.1, x)
                        -Real.1 <= x
                        arcsin(x).is_close(arcsin(x0), delta2)
                        if arcsin(x) = arcsin(x0) {
                            sin_arcsin(x)
                            (arcsin(x)).sin = x
                            sin_arcsin(x0)
                            (arcsin(x0)).sin = x0
                            arcsin(x) = arcsin(x0)
                            (arcsin(x)).sin = (arcsin(x0)).sin
                            x = x0
                            false
                        }
                        arcsin(x) != arcsin(x0)
                        difference_quotient(Real.sin, arcsin(x0), arcsin(x)).is_close(
                            (arcsin(x0)).cos, delta_r)
                        continuous_condition(reciprocal_real, (arcsin(x0)).cos, delta_r, eps) =
                            forall(x1: Real) {
                                x1.is_close((arcsin(x0)).cos, delta_r)
                                implies reciprocal_real(x1).is_close(
                                    reciprocal_real((arcsin(x0)).cos), eps)
                            }
                        reciprocal_real(difference_quotient(Real.sin, arcsin(x0), arcsin(x))).is_close(
                            reciprocal_real((arcsin(x0)).cos), eps)
                        difference_quotient(arcsin, x0, x) = (arcsin(x) - arcsin(x0)) / (x - x0)
                        difference_quotient(Real.sin, arcsin(x0), arcsin(x)) =
                            ((arcsin(x)).sin - (arcsin(x0)).sin) / (arcsin(x) - arcsin(x0))
                        sin_arcsin(x)
                        (arcsin(x)).sin = x
                        sin_arcsin(x0)
                        (arcsin(x0)).sin = x0
                        difference_quotient(Real.sin, arcsin(x0), arcsin(x)) =
                            (x - x0) / (arcsin(x) - arcsin(x0))
                        sub_ne_zero_of_ne(x, x0)
                        x - x0 != Real.0
                        sub_ne_zero_of_ne(arcsin(x), arcsin(x0))
                        arcsin(x) - arcsin(x0) != Real.0
                        inverse_div(x - x0, arcsin(x) - arcsin(x0))
                        ((x - x0) / (arcsin(x) - arcsin(x0))).inverse =
                            (arcsin(x) - arcsin(x0)) / (x - x0)
                        (difference_quotient(Real.sin, arcsin(x0), arcsin(x))).inverse =
                            (arcsin(x) - arcsin(x0)) / (x - x0)
                        reciprocal_real(difference_quotient(Real.sin, arcsin(x0), arcsin(x))) =
                            (difference_quotient(Real.sin, arcsin(x0), arcsin(x))).inverse
                        reciprocal_real(difference_quotient(Real.sin, arcsin(x0), arcsin(x))) =
                            difference_quotient(arcsin, x0, x)
                        difference_quotient(arcsin, x0, x).is_close(
                            reciprocal_real((arcsin(x0)).cos), eps)
                        reciprocal_real_apply((arcsin(x0)).cos)
                        reciprocal_real((arcsin(x0)).cos) = (arcsin(x0)).cos.inverse
                        Real.1 / (arcsin(x0)).cos = Real.1 * (arcsin(x0)).cos.inverse
                        Real.1 * (arcsin(x0)).cos.inverse = (arcsin(x0)).cos.inverse
                        Real.1 / (arcsin(x0)).cos = (arcsin(x0)).cos.inverse
                        reciprocal_real((arcsin(x0)).cos) = Real.1 / (arcsin(x0)).cos
                        difference_quotient(arcsin, x0, x).is_close(
                            Real.1 / (arcsin(x0)).cos, eps)
                    }
                }
                exists(delta4: Real) {
                    delta4.is_positive and forall(x: Real) {
                        x != x0 and x.is_close(x0, delta4)
                        implies difference_quotient(arcsin, x0, x).is_close(
                            Real.1 / (arcsin(x0)).cos, eps)
                    }
                }
            }
        }
        has_derivative_at(arcsin, x0, Real.1 / (arcsin(x0)).cos) = forall(eps: Real) {
            eps.is_positive implies exists(delta: Real) {
                delta.is_positive and forall(x: Real) {
                    x != x0 and x.is_close(x0, delta)
                    implies difference_quotient(arcsin, x0, x).is_close(
                        Real.1 / (arcsin(x0)).cos, eps)
                }
            }
        }
        if not has_derivative_at(arcsin, x0, Real.1 / (arcsin(x0)).cos) {
            not forall(eps: Real) {
                eps.is_positive implies exists(delta: Real) {
                    delta.is_positive and forall(x: Real) {
                        x != x0 and x.is_close(x0, delta)
                        implies difference_quotient(arcsin, x0, x).is_close(
                            Real.1 / (arcsin(x0)).cos, eps)
                    }
                }
            }
            let bad_eps: Real satisfy {
                bad_eps.is_positive and forall(delta: Real) {
                    not (delta.is_positive and forall(x: Real) {
                        x != x0 and x.is_close(x0, delta)
                        implies difference_quotient(arcsin, x0, x).is_close(
                            Real.1 / (arcsin(x0)).cos, bad_eps)
                    })
                }
            }
            exists(delta: Real) {
                delta.is_positive and forall(x: Real) {
                    x != x0 and x.is_close(x0, delta)
                    implies difference_quotient(arcsin, x0, x).is_close(
                        Real.1 / (arcsin(x0)).cos, bad_eps)
                }
            }
            false
        }
        has_derivative_at(arcsin, x0, Real.1 / (arcsin(x0)).cos)
    }
}

/// The bridge form of the arcsine derivative: the reciprocal of (arcsin x).cos
/// is the reciprocal of the square-root value (1 - x^2).sqrt.
theorem arcsin_derivative_sqrt_form(x: Real) {
    -Real.1 < x and x < Real.1 implies
    exists(y: Real) {
        (Real.1 - x * x).sqrt = Option.some(y) and Real.1 / (arcsin(x)).cos = Real.1 / y
    }
} by {
    if -Real.1 < x and x < Real.1 {
        lt_imp_lte(-Real.1, x)
        -Real.1 <= x
        lt_imp_lte(x, Real.1)
        x <= Real.1
        sqrt_one_sub_sq_eq_cos_arcsin(x)
        (Real.1 - x * x).sqrt = Option.some((arcsin(x)).cos)
        Real.1 / (arcsin(x)).cos = Real.1 / (arcsin(x)).cos
        exists(y: Real) {
            y = (arcsin(x)).cos and
            (Real.1 - x * x).sqrt = Option.some(y) and Real.1 / (arcsin(x)).cos = Real.1 / y
        }
        exists(y: Real) {
            (Real.1 - x * x).sqrt = Option.some(y) and Real.1 / (arcsin(x)).cos = Real.1 / y
        }
    }
}
/// The arccosine has derivative -(1/(arcsin(x0)).cos) at every point of (-1, 1).
theorem arccos_has_derivative_at(x0: Real) {
    -Real.1 < x0 and x0 < Real.1 implies
    has_derivative_at(arccos, x0, -(Real.1 / (arcsin(x0)).cos))
} by {
    if -Real.1 < x0 and x0 < Real.1 {
        constant_has_derivative_at(pi_over_two, x0)
        has_derivative_at(constant[Real, Real](pi_over_two), x0, Real.0)
        arcsin_has_derivative_at(x0)
        has_derivative_at(arcsin, x0, Real.1 / (arcsin(x0)).cos)
        derivative_pointwise_sub(constant[Real, Real](pi_over_two), arcsin, x0, Real.0,
            Real.1 / (arcsin(x0)).cos)
        has_derivative_at(
            pointwise_add(constant[Real, Real](pi_over_two), pointwise_neg(arcsin)),
            x0,
            Real.0 + -(Real.1 / (arcsin(x0)).cos))
        Real.0 + -(Real.1 / (arcsin(x0)).cos) = -(Real.1 / (arcsin(x0)).cos)
        has_derivative_at(
            pointwise_add(constant[Real, Real](pi_over_two), pointwise_neg(arcsin)),
            x0,
            -(Real.1 / (arcsin(x0)).cos))
        forall(x: Real) {
            arccos(x) = pi_over_two - arcsin(x)
            pointwise_add(constant[Real, Real](pi_over_two), pointwise_neg(arcsin), x) =
                constant[Real, Real](pi_over_two, x) + pointwise_neg(arcsin, x)
            constant[Real, Real](pi_over_two, x) = pi_over_two
            pointwise_neg(arcsin, x) = -arcsin(x)
            pointwise_add(constant[Real, Real](pi_over_two), pointwise_neg(arcsin), x) =
                pi_over_two - arcsin(x)
            arccos(x) = pointwise_add(constant[Real, Real](pi_over_two), pointwise_neg(arcsin), x)
        }
        function_extensionality(arccos,
            pointwise_add(constant[Real, Real](pi_over_two), pointwise_neg(arcsin)))
        function_eq_transport_predicate_rev(
            function(h: Real -> Real) {
                has_derivative_at(h, x0, -(Real.1 / (arcsin(x0)).cos))
            },
            arccos,
            pointwise_add(constant[Real, Real](pi_over_two), pointwise_neg(arcsin))
        )
        has_derivative_at(arccos, x0, -(Real.1 / (arcsin(x0)).cos))
    }
}
