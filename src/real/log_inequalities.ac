from order import lt_of_lt_of_lte, lt_trans, not_gt_imp_lte, not_gte_imp_lt, not_lt_imp_gte, not_lte_imp_gt
from real.exp import exp_increasing, exp_zero
from real.exp_inequalities import exp_ge_one_of_nonneg, exp_ge_one_plus_self, exp_le_one_of_nonpos, exp_lt_one_of_neg
from real.log import Real, exp_log_or_zero, exp_neg, log_one
from real.real_base import lte_add_right

numerals Real

/// The logarithm is bounded above by `x - 1` on positive reals.
theorem log_le_sub_one(x: Real, y: Real) {
    x > Real.0 and x.log = Option.some(y) implies y <= x - Real.1
} by {
    exp_ge_one_plus_self(y)
    y.exp >= Real.1 + y
    exp_log_or_zero(x, y)
    y.exp = x
    Real.1 + y <= x
    lte_add_right(Real.1 + y, x, -Real.1)
    Real.1 + y + -Real.1 <= x + -Real.1
    Real.1 + y + -Real.1 = y + Real.1 + -Real.1
    y + Real.1 + -Real.1 = y + (Real.1 + -Real.1)
    Real.1 + -Real.1 = Real.0
    y + (Real.1 + -Real.1) = y + Real.0
    y + Real.0 = y
    x + -Real.1 = x - Real.1
    y <= x - Real.1
}

/// The logarithm is bounded below by `1 - 1 / x` on positive reals.
theorem one_sub_recip_le_log(x: Real, y: Real) {
    x > Real.0 and x.log = Option.some(y) implies Real.1 - Real.1 / x <= y
} by {
    exp_ge_one_plus_self(-y)
    (-y).exp >= Real.1 + -y
    Real.1 + -y = Real.1 - y
    exp_log_or_zero(x, y)
    y.exp = x
    exp_neg(y)
    (-y).exp = Real.1 / y.exp
    Real.1 / y.exp = Real.1 / x
    (-y).exp = Real.1 / x
    Real.1 - y <= Real.1 / x
    lte_add_right(Real.1 - y, Real.1 / x, y)
    Real.1 - y + y <= Real.1 / x + y
    Real.1 - y + y = Real.1 + -y + y
    Real.1 + -y + y = Real.1 + (-y + y)
    -y + y = Real.0
    Real.1 + (-y + y) = Real.1 + Real.0
    Real.1 + Real.0 = Real.1
    Real.1 <= Real.1 / x + y
    lte_add_right(Real.1, Real.1 / x + y, -(Real.1 / x))
    Real.1 + -(Real.1 / x) <= Real.1 / x + y + -(Real.1 / x)
    Real.1 / x + y + -(Real.1 / x) = y + Real.1 / x + -(Real.1 / x)
    y + Real.1 / x + -(Real.1 / x) = y + (Real.1 / x + -(Real.1 / x))
    Real.1 / x + -(Real.1 / x) = Real.0
    y + (Real.1 / x + -(Real.1 / x)) = y + Real.0
    y + Real.0 = y
    Real.1 + -(Real.1 / x) = Real.1 - Real.1 / x
    Real.1 - Real.1 / x <= y
}

/// The logarithm is nonnegative on `[1, infinity)`.
theorem log_nonneg_of_ge_one(x: Real, y: Real) {
    x >= Real.1 and x.log = Option.some(y) implies y >= Real.0
} by {
    Real.1 > Real.0
    Real.0 < Real.1
    x >= Real.1
    Real.1 <= x
    lt_of_lt_of_lte(Real.0, Real.1, x)
    Real.0 < x
    x > Real.0
    if not y >= Real.0 {
        not_gte_imp_lt(y, Real.0)
        y < Real.0
        exp_lt_one_of_neg(y)
        y.exp < Real.1
        exp_log_or_zero(x, y)
        y.exp = x
        x < Real.1
        false
    }
}

/// The logarithm is nonpositive on positive reals at most one.
theorem log_nonpos_of_pos_le_one(x: Real, y: Real) {
    x > Real.0 and x <= Real.1 and x.log = Option.some(y) implies y <= Real.0
} by {
    if not y <= Real.0 {
        not_lte_imp_gt(y, Real.0)
        y > Real.0
        exp_increasing(Real.0, y)
        (Real.0).exp < y.exp
        exp_zero
        (Real.0).exp = Real.1
        exp_log_or_zero(x, y)
        y.exp = x
        Real.1 < x
        false
    }
}

/// The logarithm is positive above one.
theorem log_pos_of_gt_one(x: Real, y: Real) {
    x > Real.1 and x.log = Option.some(y) implies y > Real.0
} by {
    Real.1 > Real.0
    Real.0 < Real.1
    Real.1 < x
    lt_trans(Real.0, Real.1, x)
    Real.0 < x
    x > Real.0
    if not y > Real.0 {
        not_gt_imp_lte(y, Real.0)
        y <= Real.0
        exp_le_one_of_nonpos(y)
        y.exp <= Real.1
        exp_log_or_zero(x, y)
        y.exp = x
        x <= Real.1
        false
    }
}

/// The logarithm is negative between zero and one.
theorem log_neg_of_pos_lt_one(x: Real, y: Real) {
    x > Real.0 and x < Real.1 and x.log = Option.some(y) implies y < Real.0
} by {
    if not y < Real.0 {
        not_lt_imp_gte(y, Real.0)
        y >= Real.0
        exp_ge_one_of_nonneg(y)
        y.exp >= Real.1
        exp_log_or_zero(x, y)
        y.exp = x
        x >= Real.1
        false
    }
}

/// If a positive real has logarithm zero, then it is one.
theorem eq_one_of_log_eq_zero(x: Real) {
    x > Real.0 and x.log = Option.some(Real.0) implies x = Real.1
} by {
    exp_log_or_zero(x, Real.0)
    (Real.0).exp = x
    exp_zero
    (Real.0).exp = Real.1
    x = Real.1
}

/// The logarithm of a real equal to one is zero.
theorem log_eq_zero_of_eq_one(x: Real) {
    x = Real.1 implies x.log = Option.some(Real.0)
} by {
    log_one
    Real.1.log = Option.some(Real.0)
    x.log = Option.some(Real.0)
}

/// A positive real different from one has nonzero logarithm.
theorem log_ne_zero_of_pos_ne_one(x: Real, y: Real) {
    x > Real.0 and x != Real.1 and x.log = Option.some(y) implies y != Real.0
} by {
    if y = Real.0 {
        x.log = Option.some(Real.0)
        eq_one_of_log_eq_zero(x)
        x = Real.1
        false
    }
}

/// The logarithm is monotone on positive reals.
theorem log_monotone(x: Real, y: Real, lx: Real, ly: Real) {
    x > Real.0 and y > Real.0 and x <= y and x.log = Option.some(lx) and y.log = Option.some(ly)
    implies lx <= ly
} by {
    if not lx <= ly {
        not_lte_imp_gt(lx, ly)
        lx > ly
        exp_increasing(ly, lx)
        ly.exp < lx.exp
        exp_log_or_zero(x, lx)
        exp_log_or_zero(y, ly)
        lx.exp = x
        ly.exp = y
        y < x
        false
    }
}

/// The logarithm is strictly monotone on positive reals.
theorem log_strict_monotone(x: Real, y: Real, lx: Real, ly: Real) {
    x > Real.0 and y > Real.0 and x < y and x.log = Option.some(lx) and y.log = Option.some(ly)
    implies lx < ly
} by {
    if not lx < ly {
        not_lt_imp_gte(lx, ly)
        lx >= ly
        if lx = ly {
            exp_log_or_zero(x, lx)
            exp_log_or_zero(y, ly)
            lx.exp = x
            ly.exp = y
            lx.exp = ly.exp
            x = y
            false
        } else {
            ly < lx
            exp_increasing(ly, lx)
            ly.exp < lx.exp
            exp_log_or_zero(x, lx)
            exp_log_or_zero(y, ly)
            lx.exp = x
            ly.exp = y
            y < x
            false
        }
    }
}
