/// This file defines the pointwise product of sequences and proves convergence properties.
from nat import Nat, lte_trans
from real.real_seq import converges, converges_to, limit, tail_bound, eps_lt_half
from real.real_ring import Real, exists_small_mul_variant_2
from real.real_series import triangle_ineq
from real.real_base import lte_lt_trans

/// The pointwise product of two sequences of real numbers.
define prod_seq(a: Nat -> Real, b: Nat -> Real, n: Nat) -> Real {
    a(n) * b(n)
}

// =============================================================================
// HELPER LEMMAS FOR CONVERGENCE PROOF
// =============================================================================

/// LEMMA 3: Bounding absolute value when close
/// If x is close to y within eps, then |x| ≤ |y| + eps.
theorem abs_le_abs_add_eps(x: Real, y: Real, eps: Real) {
    x.is_close(y, eps) implies x.abs <= y.abs + eps
} by {
    // |x| = |(x-y) + y| <= |x-y| + |y| < eps + |y|
    ((x - y) + y).abs <= (x - y).abs + y.abs
    (x - y).abs <= eps
}

/// LEMMA 5: Multiplication preserves mixed inequalities for non-negatives
/// If a <= b, c < d, with appropriate sign conditions, then a*c < b*d.
theorem mul_le_lt_of_nonneg_pos(a: Real, b: Real, c: Real, d: Real) {
    a <= b and c < d
    and not a.is_negative and not c.is_negative
    and b.is_positive and d.is_positive
    implies
    a * c < b * d
} by {
    // Step 1: a * c <= b * c (using lte_mul_nonneg_right)

    // Step 2: b * c < b * d (using lt_mul_pos_left)

    // Step 3: Combine using transitivity
    a * c <= b * c and b * c < b * d implies a * c < b * d
}

/// LEMMA 6: Closeness of products (THE MAIN HELPER)
/// If a1 ≈ a2 and b1 ≈ b2, with bounds on |a1| and |b2|, then a1*b1 ≈ a2*b2.
theorem mul_close_from_close(a1: Real, a2: Real, b1: Real, b2: Real,
                              a_eps: Real, b_eps: Real,
                              a_bound: Real, b_bound: Real) {
    a1.is_close(a2, a_eps) and b1.is_close(b2, b_eps)
    and a1.abs <= a_bound and b2.abs <= b_bound
    and a_bound.is_positive and b_bound.is_positive
    and a_eps.is_positive and b_eps.is_positive
    implies
    (a1 * b1).is_close(a2 * b2, a_bound * b_eps + a_eps * b_bound)
} by {
    // Use algebraic identity
    let diff = a1 * b1 - a2 * b2
    diff = a1 * b1 - a1 * b2 + a1 * b2 - a2 * b2

    // Apply triangle inequality
    (a1 * (b1 - b2) + (a1 - a2) * b2).abs <= (a1 * (b1 - b2)).abs + ((a1 - a2) * b2).abs

    // Connect to diff
    a1 * (b1 - b2) = a1 * b1 - a1 * b2
    (a1 - a2) * b2 = a1 * b2 - a2 * b2
    a1 * (b1 - b2) + (a1 - a2) * b2 = a1 * b1 - a1 * b2 + (a1 * b2 - a2 * b2)
    a1 * b1 - a1 * b2 + (a1 * b2 - a2 * b2) = a1 * b1 - a1 * b2 + a1 * b2 - a2 * b2
    a1 * (b1 - b2) + (a1 - a2) * b2 = diff
    diff.abs <= (a1 * (b1 - b2)).abs + ((a1 - a2) * b2).abs

    // Apply mul_abs
    (a1 * (b1 - b2)).abs = a1.abs * (b1 - b2).abs
    ((a1 - a2) * b2).abs = (a1 - a2).abs * b2.abs

    // Use closeness conditions
    (b1 - b2).abs < b_eps

    // Apply mul_le_lt_of_nonneg_pos for first product
    a1.abs <= a_bound and (b1 - b2).abs < b_eps and not a1.abs.is_negative and not (b1 - b2).abs.is_negative and a_bound.is_positive and b_eps.is_positive implies a1.abs * (b1 - b2).abs < a_bound * b_eps
    a1.abs * (b1 - b2).abs < a_bound * b_eps

    // For the second product, we need case analysis since b2.abs <= b_bound (not strict)
    (a1 - a2).abs < a_eps
    if b2.abs.is_positive {
        (a1 - a2).abs * b2.abs < a_eps * b2.abs
        a_eps * b2.abs <= a_eps * b_bound
        (a1 - a2).abs * b2.abs < a_eps * b_bound
    } else {
        not b2.abs.is_negative
        b2.abs = Real.0
        (a1 - a2).abs * b2.abs = Real.0
        Real.0 < a_eps * b_bound
        (a1 - a2).abs * b2.abs < a_eps * b_bound
    }

    // Combine the bounds
    (a1 * (b1 - b2)).abs < a_bound * b_eps
    ((a1 - a2) * b2).abs < a_eps * b_bound
    (a1 * (b1 - b2)).abs + ((a1 - a2) * b2).abs < a_bound * b_eps + a_eps * b_bound

    // Apply transitivity
    diff.abs < a_bound * b_eps + a_eps * b_bound
    (a1 * b1 - a2 * b2).abs < a_bound * b_eps + a_eps * b_bound
}

// =============================================================================
// MAIN CONVERGENCE THEOREM
// =============================================================================

/// The product of two convergent sequences converges to the product of their limits.
theorem limit_prod_seq(a: Nat -> Real, b: Nat -> Real) {
    converges(a) and converges(b)
    implies
    converges_to(prod_seq(a, b), limit(a) * limit(b))
} by {
    let q = prod_seq(a, b)
    let ql = limit(a) * limit(b)

    forall(eps: Real) {
        if eps.is_positive {
            // Get bounds for limit(a) and limit(b)
            let a_abs_bound: Real satisfy {
                limit(a).abs < a_abs_bound
            }
            let a_bound = a_abs_bound + Real.1
            a_bound.is_positive

            let b_abs_bound: Real satisfy {
                limit(b).abs < b_abs_bound
            }
            let b_bound = b_abs_bound + Real.1
            b_bound.is_positive

            // Find small_eps such that a_bound * small_eps + small_eps * b_bound < eps
            let sum_bound = a_bound + b_bound
            sum_bound.is_positive
            (sum_bound + sum_bound).is_positive

            Real.1.is_positive
            let eps_half: Real satisfy {
                eps_half.is_positive and eps_half + eps_half < Real.1
            }

            // Prove existence of small eps_candidate
            let eps_candidate: Real satisfy {
                eps_candidate.is_positive and
                eps_candidate * (sum_bound + sum_bound) < eps
            }

            let small_eps = eps_candidate.min(eps_half)
            small_eps.is_positive
            small_eps + small_eps <= eps_half + eps_half
            small_eps + small_eps < Real.1
            small_eps < Real.1
            small_eps * (sum_bound + sum_bound) <= eps_candidate * (sum_bound + sum_bound) and eps_candidate * (sum_bound + sum_bound) < eps implies small_eps * (sum_bound + sum_bound) < eps

            // Get convergence points
            let n1: Nat satisfy {
                tail_bound(a, limit(a), n1, small_eps)
            }

            let n2: Nat satisfy {
                tail_bound(b, limit(b), n2, small_eps)
            }

            let n: Nat satisfy {
                n1 <= n and n2 <= n
            }

            forall(i: Nat) {
                if n <= i {
                    // Derive is_close from tail_bound
                    n1 <= i
                    n2 <= i
                    a(i).is_close(limit(a), small_eps)
                    b(i).is_close(limit(b), small_eps)

                    // Bound a(i) using abs_le_abs_add_eps
                    a(i).is_close(limit(a), small_eps) implies a(i).abs <= limit(a).abs + small_eps
                    a_abs_bound + small_eps < a_abs_bound + Real.1
                    a(i).abs < a_abs_bound + small_eps
                    a(i).abs <= a_abs_bound + small_eps and a_abs_bound + small_eps < a_bound implies a(i).abs < a_bound

                    // Bound limit(b)
                    // Apply mul_close_from_close
                    limit(b).abs <= b_bound

                    a(i).is_close(limit(a), small_eps) and b(i).is_close(limit(b), small_eps) and a(i).abs <= a_bound and limit(b).abs <= b_bound and a_bound.is_positive and b_bound.is_positive and small_eps.is_positive and small_eps.is_positive implies (a(i) * b(i)).is_close(limit(a) * limit(b), a_bound * small_eps + small_eps * b_bound)

                    (a(i) * b(i)).is_close(
                        limit(a) * limit(b),
                        a_bound * small_eps + small_eps * b_bound)

                    // Show this is < eps
                    // Algebra: a_bound * small_eps + small_eps * b_bound = small_eps * (a_bound + b_bound) = small_eps * sum_bound
                    a_bound * small_eps = small_eps * a_bound
                    small_eps * a_bound + small_eps * b_bound = small_eps * (a_bound + b_bound)
                    a_bound * small_eps + small_eps * b_bound = small_eps * (a_bound + b_bound)
                    a_bound + b_bound = sum_bound
                    a_bound * small_eps + small_eps * b_bound = small_eps * sum_bound
                    // Now show small_eps * sum_bound < small_eps * (sum_bound + sum_bound)
                    sum_bound < sum_bound + sum_bound
                    small_eps * sum_bound < small_eps * (sum_bound + sum_bound)
                    a_bound * small_eps + small_eps * b_bound < small_eps * (sum_bound + sum_bound)
                    small_eps * (sum_bound + sum_bound) < eps
                    a_bound * small_eps + small_eps * b_bound < eps

                    // Conclude
                    (a(i) * b(i) - (limit(a) * limit(b))).abs < a_bound * small_eps + small_eps * b_bound
                    (a(i) * b(i) - (limit(a) * limit(b))).abs < eps
                    q(i).is_close(ql, eps)
                }
            }
            exists(k0: Nat) {
                n <= k0 and not q(k0).is_close(ql, eps)
            } or tail_bound(q, ql, n, eps)
            tail_bound(q, ql, n, eps)
        }
    }
    converges_to(q, ql)
}

/// The product of two convergent sequences converges.
theorem prod_seq_converges(a: Nat -> Real, b: Nat -> Real) {
    converges(a) and converges(b)
    implies
    converges(prod_seq(a, b))
} by {
}

/// The limit of the product is the product of the limits.
theorem limit_of_prod(a: Nat -> Real, b: Nat -> Real) {
    converges(a) and converges(b)
    implies
    limit(prod_seq(a, b)) = limit(a) * limit(b)
} by {
    converges_to(prod_seq(a, b), limit(a) * limit(b))
}
