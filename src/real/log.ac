from nat import Nat, alt_induction
from rat import Rat
from list import partial
from order import lte_antisymm, lte_trans, lt_of_lt_of_lte, not_lt_imp_gte, not_lte_imp_gt
from nat import from_nat, from_nat_zero, from_nat_add
from data.basic.set import Set
from data.basic.witness import choose_or_default, choose_or_default_spec, choose_or_default_unique_eq, exists_unique, exists_unique_intro
from real.exp import Real, exp_add, exp_term, exp_term_partial_converges, div_lt_div_pos, exp_unbounded, exp_zero, exp_pos, exp_increasing, factorial_pos
from real.harmonic import rat_from_nat_lte_of_nat_lte, real_inverse_antitone_pos_strict, real_one_div_pos, real_recip_antitone_pos
from real.real_base import lt_add_converse, gt_zero_imp_pos
from real.real_field import mul_div_cancel
from real.real_ring import converges, limit, lt_mul_pos_left
from real.real_seq import eps_lt_half
from real.real_series import geom_converges, geom_series_no_div, pow_nonneg, seq_lte, seq_lte_preserves_limit
from real.supremum import completeness, has_upper_bound, is_nonempty, is_set_supremum, is_set_upper_bound

numerals Real

/// True if `y` is a logarithm of `x` with respect to the real exponential.
define is_log(x: Real, y: Real) -> Bool {
    y.exp = x
}

/// The real logarithm, defined exactly on positive reals.
attributes Real {
    /// The real logarithm, defined exactly on positive reals.
    define log(self) -> Option[Real] {
        if self > Real.0 {
            Option.some(choose_or_default(is_log(self), Real.0))
        } else {
            Option.none[Real]
        }
    }
}

/// The logarithm value, defaulting to zero outside the positive real domain.
define log_value(x: Real) -> Real {
    option_get_or_else(x.log, Real.0)
}


/// True if `y` lies in the lower exponential preimage of `x`.
define exp_lower_preimage_contains(x: Real, y: Real) -> Bool {
    y.exp <= x
}

/// The set of real `y` such that `y.exp <= x`.
define exp_lower_preimage(x: Real) -> Set[Real] {
    Set[Real].new(exp_lower_preimage_contains(x))
}

/// Positive exponential values are nonzero.
theorem exp_ne_zero(x: Real) {
    x.exp != Real.0
} by {
    exp_pos(x)
    x.exp > Real.0
}

/// The exponential of a negated argument is the reciprocal exponential.
theorem exp_neg(x: Real) {
    (-x).exp = Real.1 / x.exp
} by {
    exp_add(x, -x)
    x + -x = Real.0
    (Real.0).exp = x.exp * (-x).exp
    exp_zero
    (Real.0).exp = Real.1
    x.exp * (-x).exp = Real.1
    x.exp != Real.0
    (-x).exp = Real.1 / x.exp
}

/// Equal exponential values have equal arguments.
theorem exp_injective(x: Real, y: Real) {
    x.exp = y.exp implies x = y
} by {
    if x.exp = y.exp {
        if x < y {
            exp_increasing(x, y)
            x.exp < y.exp
            false
        }
        if y < x {
            exp_increasing(y, x)
            y.exp < x.exp
            false
        }
        x >= y
        y >= x
        lte_antisymm(x, y)
        x = y
    }
}

/// The exponential relation gives a unique logarithm.
theorem is_log_unique(x: Real, y: Real, z: Real) {
    is_log(x, y) and is_log(x, z) implies y = z
} by {
    if is_log(x, y) and is_log(x, z) {
        y.exp = x
        z.exp = x
        y.exp = z.exp
        exp_injective(y, z)
        y = z
    }
}

/// A real number in the image of the exponential has a unique logarithm.
theorem exists_unique_log_of_exp(x: Real) {
    exists_unique(is_log(x.exp))
} by {
    is_log(x.exp, x)
    forall(y: Real) {
        if is_log(x.exp, y) {
            y.exp = x.exp
            exp_injective(y, x)
            y = x
        }
    }
    exists_unique_intro(is_log(x.exp), x)
    exists_unique(is_log(x.exp))
}

/// The logarithm is a left inverse to the exponential.
theorem log_exp(x: Real) {
    (x.exp).log = Option.some(x)
} by {
    exp_pos(x)
    x.exp > Real.0
    exists_unique_log_of_exp(x)
    is_log(x.exp, x)
    (x.exp).log = Option.some(x)
    option_get_or_else_some[Real](x, Real.0)
    option_get_or_else(Option.some(x), Real.0) = x
    (x.exp).log.get_or_else(Real.0) = x
    (x.exp).log = Option.some((x.exp).log.get_or_else(Real.0))
}

/// The defaulting logarithm witness exponentiates back to `x` whenever an exponential preimage exists.
theorem exp_log_value_of_exists(x: Real) {
    exists(y: Real) {
        y.exp = x
    } implies (x.log.get_or_else(Real.0)).exp = x
} by {
    if exists(y: Real) { y.exp = x } {
        exists(y: Real) {
            is_log(x, y)
        }
        let y: Real satisfy {
            is_log(x, y)
        }
        is_log(x, y)
        y.exp = x
        option_get_or_else_some[Real](y, Real.0)
        option_get_or_else(Option.some(y), Real.0) = y
        x.log.get_or_else(Real.0) = y
        (x.log.get_or_else(Real.0)).exp = x
    }
}

/// Exponential values are recovered by exponentiating their logarithm.
theorem exp_log_exp(x: Real) {
    (x.exp).log = Option.some(x)
} by {
    log_exp(x)
}

/// The exponential of a natural multiple is the natural power of the exponential.
theorem exp_nat_mul(x: Real, n: Nat) {
    (from_nat[Real](n) * x).exp = x.exp.pow(n)
} by {
    define p(k: Nat) -> Bool {
        (from_nat[Real](k) * x).exp = x.exp.pow(k)
    }

    from_nat[Real](Nat.0) = Real.0
    Real.0 * x = Real.0
    (Real.0 * x).exp = (Real.0).exp
    exp_zero
    (Real.0).exp = Real.1
    x.exp.pow(Nat.0) = Real.1
    p(Nat.0)

    forall(k: Nat) {
        if p(k) {
            from_nat_add[Real](k, Nat.1)
            from_nat[Real](k + Nat.1) = from_nat[Real](k) + from_nat[Real](Nat.1)
            k + Nat.1 = k.suc
            from_nat[Real](Nat.1) = Real.1
            from_nat[Real](k.suc) = from_nat[Real](k) + Real.1
            from_nat[Real](k.suc) * x = (from_nat[Real](k) + Real.1) * x
            (from_nat[Real](k) + Real.1) * x = from_nat[Real](k) * x + x
            (from_nat[Real](k.suc) * x).exp = (from_nat[Real](k) * x + x).exp
            exp_add(from_nat[Real](k) * x, x)
            (from_nat[Real](k) * x + x).exp = (from_nat[Real](k) * x).exp * x.exp
            (from_nat[Real](k) * x).exp = x.exp.pow(k)
            (from_nat[Real](k.suc) * x).exp = x.exp.pow(k) * x.exp
            x.exp.pow(k.suc) = x.exp * x.exp.pow(k)
            x.exp.pow(k) * x.exp = x.exp * x.exp.pow(k)
            (from_nat[Real](k.suc) * x).exp = x.exp.pow(k.suc)
            p(k.suc)
        }
    }

    alt_induction(p)
    forall(k: Nat) { p(k) }
    p(n)
}

/// Each nonnegative exponential-series term is bounded by the matching geometric term.
theorem exp_term_le_pow_of_nonneg(x: Real, n: Nat) {
    Real.0 <= x implies exp_term(x, n) <= x.pow(n)
} by {
    if Real.0 <= x {
        let denom = Real.from_rat(Rat.from_nat(n.factorial))
        factorial_pos(n)
        denom > Real.0
        Nat.1 <= n.factorial
        rat_from_nat_lte_of_nat_lte(Nat.1, n.factorial)
        Rat.1 <= Rat.from_nat(n.factorial)
        Real.from_rat(Rat.1) <= Real.from_rat(Rat.from_nat(n.factorial))
        Real.from_rat(Rat.1) = Real.1
        denom >= Real.1
        pow_nonneg(x, n)
        Real.0 <= x.pow(n)
        exp_term(x, n) = x.pow(n) / denom
        Real.1 > Real.0
        real_recip_antitone_pos(Real.1, denom)
        Real.1 / denom <= Real.1 / Real.1
        Real.1 / Real.1 = Real.1
        Real.1 / denom <= Real.1
        x.pow(n) * (Real.1 / denom) <= x.pow(n) * Real.1
        x.pow(n) / denom = x.pow(n) * (Real.1 / denom)
        x.pow(n) * Real.1 = x.pow(n)
        x.pow(n) / denom <= x.pow(n)
        exp_term(x, n) <= x.pow(n)
    }
}

/// For `0 <= x < 1`, the exponential is bounded by the geometric series.
theorem exp_le_geometric_limit(x: Real) {
    Real.0 <= x and x < Real.1 implies x.exp <= limit(partial(x.pow))
} by {
    if Real.0 <= x and x < Real.1 {
        forall(n: Nat) {
            exp_term_le_pow_of_nonneg(x, n)
            exp_term(x, n) <= x.pow(n)
        }
        seq_lte(exp_term(x), x.pow)
        exp_term_partial_converges(x)
        converges(partial(exp_term(x)))
        x.abs = x
        x.abs < Real.1
        geom_converges(x)
        converges(partial(x.pow))
        seq_lte_preserves_limit(exp_term(x), x.pow)
        limit(partial(exp_term(x))) <= limit(partial(x.pow))
        x.exp = limit(partial(exp_term(x)))
        x.exp <= limit(partial(x.pow))
    }
}

/// The nonnegative geometric series has sum `1 / (1 - x)` for `x < 1`.
theorem geometric_limit_eq_recip(x: Real) {
    Real.0 <= x and x < Real.1 implies limit(partial(x.pow)) = Real.1 / (Real.1 - x)
} by {
    if Real.0 <= x and x < Real.1 {
        x.abs = x
        x.abs < Real.1
        geom_series_no_div(x)
        let geom = limit(partial(x.pow))
        Real.1 + x * geom = geom
        geom - x * geom = Real.1
        (Real.1 - x) * geom = Real.1
        Real.1 - x > Real.0
        Real.1 - x != Real.0
        geom = Real.1 / (Real.1 - x)
        limit(partial(x.pow)) = Real.1 / (Real.1 - x)
    }
}

/// The exponential is bounded by the closed-form geometric estimate.
theorem exp_le_geometric_recip(x: Real) {
    Real.0 <= x and x < Real.1 implies x.exp <= Real.1 / (Real.1 - x)
} by {
    if Real.0 <= x and x < Real.1 {
        exp_le_geometric_limit(x)
        geometric_limit_eq_recip(x)
        x.exp <= limit(partial(x.pow))
        limit(partial(x.pow)) = Real.1 / (Real.1 - x)
        x.exp <= Real.1 / (Real.1 - x)
    }
}

/// Exponential values can be made arbitrarily close to `1` from above.
theorem exists_small_exp_below(r: Real) {
    r > Real.1 implies exists(d: Real) {
        d > Real.0 and d.exp < r
    }
} by {
    if r > Real.1 {
        Real.1 > Real.0
        Real.0 < Real.1
        Real.1 < r
        Real.0 < r
        r > Real.0
        r != Real.0
        div_lt_div_pos(Real.1, r, r)
        Real.1 / r < r / r
        r / r = Real.1
        Real.1 / r < Real.1
        real_one_div_pos(r)
        Real.1 / r > Real.0
        let gap = Real.1 - Real.1 / r
        gap > Real.0
        eps_lt_half(gap)
        let d: Real satisfy {
            d.is_positive and d + d < gap
        }
        d > Real.0
        d + d < gap
        d < d + d
        d < gap
        gap = Real.1 - Real.1 / r
        Real.1 - Real.1 / r < Real.1 - Real.0
        Real.1 - Real.0 = Real.1
        Real.1 - Real.1 / r < Real.1
        gap < Real.1
        d < Real.1
        Real.0 <= d
        exp_le_geometric_recip(d)
        d.exp <= Real.1 / (Real.1 - d)
        d < Real.1 - Real.1 / r
        d + Real.1 / r < (Real.1 - Real.1 / r) + Real.1 / r
        (Real.1 - Real.1 / r) + Real.1 / r = Real.1
        d + Real.1 / r < Real.1
        Real.1 / r + d = d + Real.1 / r
        (Real.1 - d) + d = Real.1
        Real.1 / r + d < (Real.1 - d) + d
        lt_add_converse(Real.1 / r, Real.1 - d, d)
        Real.1 / r < Real.1 - d
        Real.1 / r > Real.0
        Real.1 - d > Real.0
        real_inverse_antitone_pos_strict(Real.1 / r, Real.1 - d)
        (Real.1 - d).inverse < (Real.1 / r).inverse
        Real.1 / (Real.1 - d) = (Real.1 - d).inverse
        (Real.1 / r).inverse = r
        Real.1 / (Real.1 - d) < r
        if not d.exp < r {
            d.exp >= r
            r <= d.exp
            Real.1 / (Real.1 - d) < r
            exp_le_geometric_recip(d)
            d.exp <= Real.1 / (Real.1 - d)
            lte_trans(r, d.exp, Real.1 / (Real.1 - d))
            r <= Real.1 / (Real.1 - d)
            false
        }
        not d.exp >= r
        d.exp < r
        exists(z: Real) {
            z > Real.0 and z.exp < r
        }
    }
}

/// The lower exponential preimage of a positive real number is nonempty.
theorem exp_lower_preimage_nonempty(x: Real) {
    x > Real.0 implies is_nonempty(exp_lower_preimage(x))
} by {
    if x > Real.0 {
        if x >= Real.1 {
            exp_zero
            (Real.0).exp = Real.1
            (Real.0).exp <= x
            exp_lower_preimage(x).contains(Real.0)
            is_nonempty(exp_lower_preimage(x))
        } else {
            x < Real.1
            real_one_div_pos(x)
            Real.1 / x > Real.0
            exp_unbounded(Real.1 / x)
            let b: Real satisfy {
                Real.1 / x < b.exp
            }
            exp_pos(b)
            b.exp > Real.0
            real_inverse_antitone_pos_strict(Real.1 / x, b.exp)
            b.exp.inverse < (Real.1 / x).inverse
            exp_neg(b)
            (-b).exp = Real.1 / b.exp
            Real.1 / b.exp = b.exp.inverse
            (-b).exp = b.exp.inverse
            (Real.1 / x).inverse = x
            b.exp.inverse < x
            (-b).exp < x
            (-b).exp <= x
            exp_lower_preimage(x).contains(-b)
            is_nonempty(exp_lower_preimage(x))
        }
    }
}

/// The lower exponential preimage of a positive real number is bounded above.
theorem exp_lower_preimage_has_upper_bound(x: Real) {
    x > Real.0 implies has_upper_bound(exp_lower_preimage(x))
} by {
    if x > Real.0 {
        exp_unbounded(x)
        let b: Real satisfy {
            x < b.exp
        }
        forall(y: Real) {
            if exp_lower_preimage(x).contains(y) {
                y.exp <= x
                if b < y {
                    exp_increasing(b, y)
                    b.exp < y.exp
                    b.exp <= y.exp
                    lte_trans(b.exp, y.exp, x)
                    b.exp <= x
                    false
                }
                y <= b
            }
        }
        is_set_upper_bound(exp_lower_preimage(x), b)
        has_upper_bound(exp_lower_preimage(x))
    }
}

/// Every positive real number is an exponential value.
theorem exists_log_of_pos(x: Real) {
    x > Real.0 implies exists(y: Real) {
        y.exp = x
    }
} by {
    if x > Real.0 {
        let s = exp_lower_preimage(x)
        exp_lower_preimage_nonempty(x)
        is_nonempty(s)
        exp_lower_preimage_has_upper_bound(x)
        has_upper_bound(s)
        completeness(s)
        let y: Real satisfy {
            is_set_supremum(s, y)
        }
        is_set_supremum(s, y)
        is_set_upper_bound(s, y)

        if y.exp < x {
            exp_pos(y)
            y.exp > Real.0
            let ratio = x / y.exp
            div_lt_div_pos(y.exp, x, y.exp)
            y.exp / y.exp < x / y.exp
            y.exp / y.exp = Real.1
            ratio > Real.1
            exists_small_exp_below(ratio)
            let d: Real satisfy {
                d > Real.0 and d.exp < ratio
            }
            d > Real.0
            d.exp < ratio
            y.exp > Real.0
            y.exp.is_positive
            lt_mul_pos_left(y.exp, d.exp, ratio)
            y.exp * d.exp < y.exp * ratio
            y.exp * ratio = y.exp * (x / y.exp)
            mul_div_cancel(x, y.exp)
            y.exp * (x / y.exp) = x
            y.exp * d.exp < x
            exp_add(y, d)
            (y + d).exp = y.exp * d.exp
            (y + d).exp < x
            (y + d).exp <= x
            exp_lower_preimage_contains(x, y + d)
            exp_lower_preimage(x).contains(y + d)
            s.contains(y + d)
            is_set_upper_bound(s, y) = forall(z: Real) {
                s.contains(z) implies z <= y
            }
            y + d <= y
            y < y + d
            false
        }

        if x < y.exp {
            exp_pos(y)
            y.exp > Real.0
            let ratio = y.exp / x
            div_lt_div_pos(x, y.exp, x)
            x / x < y.exp / x
            x / x = Real.1
            ratio > Real.1
            exists_small_exp_below(ratio)
            let d: Real satisfy {
                d > Real.0 and d.exp < ratio
            }
            d > Real.0
            d.exp < ratio
            exp_pos(d)
            d.exp > Real.0
            x > Real.0
            x.is_positive
            lt_mul_pos_left(x, d.exp, ratio)
            x * d.exp < x * ratio
            x * ratio = x * (y.exp / x)
            mul_div_cancel(y.exp, x)
            x * (y.exp / x) = y.exp
            x * d.exp < y.exp
            exp_add(y - d, d)
            y - d + d = y
            y.exp = (y - d).exp * d.exp
            if not x < (y - d).exp {
                not_lt_imp_gte(x, (y - d).exp)
                x >= (y - d).exp
                if not (y - d).exp <= x {
                    not_lte_imp_gt((y - d).exp, x)
                    (y - d).exp > x
                    x < (y - d).exp
                    false
                }
                (y - d).exp <= x
                (y - d).exp * d.exp <= x * d.exp
                y.exp <= x * d.exp
                false
            }
            x < (y - d).exp
            forall(z: Real) {
                if s.contains(z) {
                    s.contains(z) = exp_lower_preimage(x).contains(z)
                    exp_lower_preimage(x).contains(z) = z.exp <= x
                    z.exp <= x
                    if y - d < z {
                        exp_increasing(y - d, z)
                        (y - d).exp < z.exp
                        z.exp <= x
                        lt_of_lt_of_lte((y - d).exp, z.exp, x)
                        (y - d).exp < x
                        false
                    }
                    z <= y - d
                }
            }
            is_set_upper_bound(s, y - d)
            is_set_supremum(s, y) = is_set_upper_bound(s, y) and forall(b: Real) {
                is_set_upper_bound(s, b) implies y <= b
            }
            y <= y - d
            y - d + d = y
            y - d < y - d + d
            y - d < y
            false
        }

        y.exp >= x
        y.exp <= x
        lte_antisymm(y.exp, x)
        y.exp = x
        exists(z: Real) {
            z.exp = x
        }
    }
}

/// A positive input's logarithm value satisfies the is_log relation.
theorem log_some_of_pos(x: Real, y: Real) {
    x > Real.0 and x.log = Option.some(y) implies is_log(x, y)
} by {
    if x > Real.0 and x.log = Option.some(y) {
        x.log = Option.some(choose_or_default(is_log(x), Real.0))
        Option.some(y) = Option.some(choose_or_default(is_log(x), Real.0))
        some_injective[Real](y, choose_or_default(is_log(x), Real.0))
        y = choose_or_default(is_log(x), Real.0)
        exists_log_of_pos(x)
        exists(z: Real) {
            is_log(x, z)
        }
        choose_or_default_spec(is_log(x), Real.0)
        is_log(x, choose_or_default(is_log(x), Real.0))
        is_log(x, y)
    }
}

/// A positive input has some logarithm value.
theorem log_some_of_pos_exists(x: Real) {
    x > Real.0 implies exists(y: Real) {
        x.log = Option.some(y)
    }
} by {
    if x > Real.0 {
        x.log = Option.some(choose_or_default(is_log(x), Real.0))
        exists(witness: Real) {
            witness = choose_or_default(is_log(x), Real.0) and
            x.log = Option.some(witness)
        }
    }
}

/// Nonpositive inputs have no logarithm value.
theorem log_none_of_nonpos(x: Real) {
    not x > Real.0 implies x.log = Option.none[Real]
} by {
    if not x > Real.0 {
        x.log = Option.none[Real]
    }
}

/// A logarithm value of a positive input exponentiates back to the input.
theorem exp_log_or_zero(x: Real, y: Real) {
    x > Real.0 and x.log = Option.some(y) implies y.exp = x
} by {
    if x > Real.0 and x.log = Option.some(y) {
        log_some_of_pos(x, y)
        is_log(x, y)
        y.exp = x
    }
}

/// The logarithm of one is zero.
theorem log_one {
    Real.1.log = Option.some(Real.0)
} by {
    exp_zero
    (Real.0).exp = Real.1
    log_exp(Real.0)
    ((Real.0).exp).log = Option.some(Real.0)
    Real.1.log = Option.some(Real.0)
}

/// The logarithm of a positive product is the sum of the logarithms.
theorem log_mul(x: Real, y: Real, a: Real, b: Real) {
    x > Real.0 and y > Real.0 and x.log = Option.some(a) and y.log = Option.some(b)
    implies (x * y).log = Option.some(a + b)
} by {
    if x > Real.0 and y > Real.0 and x.log = Option.some(a) and y.log = Option.some(b) {
        exp_log_or_zero(x, a)
        exp_log_or_zero(y, b)
        a.exp = x
        b.exp = y
        exp_add(a, b)
        (a + b).exp = a.exp * b.exp
        (a + b).exp = x * y
        (x * y) > Real.0
        (x * y).log = Option.some(a + b)
    }
}

/// Real exponentiation where it is defined over the reals.
/// Positive bases use `(exponent * base.log_or_zero).exp`; `0^0 = 1`; and `0^x = 0` for positive `x`.
attributes Real {
    /// Real exponentiation where it is defined over the reals.
    /// Positive bases use `(exponent * self.log_or_zero).exp`; `0^0 = 1`; and `0^x = 0` for positive `x`.
    define rpow(self, exponent: Real) -> Option[Real] {
        if self > Real.0 {
            Option.some((exponent * self.log.get_or_else(Real.0)).exp)
        } else {
            if self = Real.0 {
                if exponent = Real.0 {
                    Option.some(Real.1)
                } else {
                    if exponent > Real.0 {
                        Option.some(Real.0)
                    } else {
                        Option.none[Real]
                    }
                }
            } else {
                Option.none[Real]
            }
        }
    }
}

/// Real powers with positive base are positive.
theorem rpow_pos(base: Real, exponent: Real) {
    base > Real.0 implies exists(value: Real) {
        base.rpow(exponent) = Option.some(value) and value > Real.0
    }
} by {
    if base > Real.0 {
        base.rpow(exponent) = Option.some((exponent * base.log.get_or_else(Real.0)).exp)
        exp_pos(exponent * base.log.get_or_else(Real.0))
        (exponent * base.log.get_or_else(Real.0)).exp > Real.0
        exists(value: Real) {
            value = (exponent * base.log.get_or_else(Real.0)).exp and
            base.rpow(exponent) = Option.some(value) and value > Real.0
        }
    }
}

/// The zeroth real power of a nonnegative base is one.
theorem rpow_zero(base: Real) {
    base >= Real.0 implies base.rpow(Real.0) = Option.some(Real.1)
} by {
    if base > Real.0 {
        base.rpow(Real.0) = Option.some((Real.0 * base.log.get_or_else(Real.0)).exp)
        Real.0 * base.log.get_or_else(Real.0) = Real.0
        exp_zero
        (Real.0).exp = Real.1
        base.rpow(Real.0) = Option.some(Real.1)
    } else {
        base = Real.0
        base.rpow(Real.0) = Option.some(Real.1)
    }
}

/// Negative bases are outside the real-power domain used here.
theorem rpow_neg_base(base: Real, exponent: Real) {
    base < Real.0 implies base.rpow(exponent) = Option.none[Real]
} by {
    if base < Real.0 {
        not base > Real.0
        base != Real.0
        base.rpow(exponent) = Option.none[Real]
    }
}

/// Zero to a positive exponent is zero.
theorem rpow_zero_base_pos(exponent: Real) {
    exponent > Real.0 implies (Real.0).rpow(exponent) = Option.some(Real.0)
} by {
    if exponent > Real.0 {
        not Real.0 > Real.0
        exponent != Real.0
        (Real.0).rpow(exponent) = Option.some(Real.0)
    }
}

/// Zero to the zeroth power is one.
theorem rpow_zero_zero {
    (Real.0).rpow(Real.0) = Option.some(Real.1)
} by {
    not Real.0 > Real.0
    (Real.0).rpow(Real.0) = Option.some(Real.1)
}

/// The first real power of a positive base is the base.
theorem rpow_one(base: Real) {
    base > Real.0 implies base.rpow(Real.1) = Option.some(base)
} by {
    if base > Real.0 {
        base.rpow(Real.1) = Option.some((Real.1 * base.log.get_or_else(Real.0)).exp)
        Real.1 * base.log.get_or_else(Real.0) = base.log.get_or_else(Real.0)
        log_some_of_pos_exists(base)
        let y: Real satisfy {
            base.log = Option.some(y)
        }
        exp_log_or_zero(base, y)
        y.exp = base
        (base.log.get_or_else(Real.0)).exp = base
        base.rpow(Real.1) = Option.some(base)
    }
}

/// Real powers of a fixed positive base turn addition into multiplication.
theorem rpow_add(base: Real, a: Real, b: Real) {
    base > Real.0 implies base.rpow(a + b) = Option.some((a * base.log.get_or_else(Real.0)).exp * (b * base.log.get_or_else(Real.0)).exp)
} by {
    if base > Real.0 {
        base.rpow(a + b) = Option.some(((a + b) * base.log.get_or_else(Real.0)).exp)
        (a + b) * base.log.get_or_else(Real.0) = a * base.log.get_or_else(Real.0) + b * base.log.get_or_else(Real.0)
        exp_add(a * base.log.get_or_else(Real.0), b * base.log.get_or_else(Real.0))
        (a * base.log.get_or_else(Real.0) + b * base.log.get_or_else(Real.0)).exp = (a * base.log.get_or_else(Real.0)).exp * (b * base.log.get_or_else(Real.0)).exp
        base.rpow(a + b) = Option.some((a * base.log.get_or_else(Real.0)).exp * (b * base.log.get_or_else(Real.0)).exp)
    }
}

/// The logarithm of a positive-base real power is the exponent times the base logarithm.
theorem log_rpow(base: Real, exponent: Real) {
    base > Real.0 implies ((exponent * base.log.get_or_else(Real.0)).exp).log = Option.some(exponent * base.log.get_or_else(Real.0))
} by {
    if base > Real.0 {
        log_exp(exponent * base.log.get_or_else(Real.0))
        ((exponent * base.log.get_or_else(Real.0)).exp).log = Option.some(exponent * base.log.get_or_else(Real.0))
    }
}

/// Real powers multiply exponents on positive bases.
theorem rpow_mul(base: Real, a: Real, b: Real) {
    base > Real.0 implies base.rpow(a * b) = Option.some((b * (a * base.log.get_or_else(Real.0))).exp)
} by {
    if base > Real.0 {
        base.rpow(a * b) = Option.some(((a * b) * base.log.get_or_else(Real.0)).exp)
        b * (a * base.log.get_or_else(Real.0)) = (a * b) * base.log.get_or_else(Real.0)
        base.rpow(a * b) = Option.some((b * (a * base.log.get_or_else(Real.0))).exp)
    }
}

/// Natural real exponents agree with the ordinary natural power on exponential values.
theorem rpow_exp_nat(x: Real, n: Nat) {
    (x.exp).rpow(from_nat[Real](n)) = Option.some(x.exp.pow(n))
} by {
    exp_pos(x)
    log_exp(x)
    (x.exp).rpow(from_nat[Real](n)) = Option.some((from_nat[Real](n) * (x.exp).log.get_or_else(Real.0)).exp)
    (x.exp).log = Option.some(x)
    (x.exp).log = Option.some((x.exp).log.get_or_else(Real.0))
    Option.some((x.exp).log.get_or_else(Real.0)) = Option.some(x)
    some_injective[Real]((x.exp).log.get_or_else(Real.0), x)
    (x.exp).log.get_or_else(Real.0) = x
    (from_nat[Real](n) * (x.exp).log.get_or_else(Real.0)).exp = (from_nat[Real](n) * x).exp
    exp_nat_mul(x, n)
    (from_nat[Real](n) * x).exp = x.exp.pow(n)
    (x.exp).rpow(from_nat[Real](n)) = Option.some(x.exp.pow(n))
}

/// Natural real exponents agree with ordinary natural powers for positive bases.
theorem rpow_nat(base: Real, n: Nat) {
    base > Real.0 implies base.rpow(from_nat[Real](n)) = Option.some(base.pow(n))
} by {
    if base > Real.0 {
        log_some_of_pos_exists(base)
        let y: Real satisfy {
            base.log = Option.some(y)
        }
        exp_log_or_zero(base, y)
        y.exp = base
        (base.log.get_or_else(Real.0)).exp = base
        base.rpow(from_nat[Real](n)) = Option.some((from_nat[Real](n) * base.log.get_or_else(Real.0)).exp)
        exp_nat_mul(base.log.get_or_else(Real.0), n)
        (from_nat[Real](n) * base.log.get_or_else(Real.0)).exp = (base.log.get_or_else(Real.0)).exp.pow(n)
        (base.log.get_or_else(Real.0)).exp.pow(n) = base.pow(n)
        base.rpow(from_nat[Real](n)) = Option.some(base.pow(n))
    }
}
