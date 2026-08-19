/// Wallis' product for pi.
///
/// Wallis' product is the infinite product
///     pi / 2 = prod_{k=1}^infinity (2k/(2k-1)) * (2k/(2k+1)).
/// The library has no infinite-product machinery (real_series.ac and
/// abs_conv.ac only handle products as pointwise products of sequences and
/// Cauchy products of series), so the product is expressed as the limit of
/// its partial products prod_{k=1}^n (2k/(2k-1)) * (2k/(2k+1)) as n tends to
/// infinity.  This file proves the partial-product evaluation: the n-th
/// partial product telescopes to
///     (2*4*...*2n)^2 / ((1*3*...*(2n-1)) * (3*5*...*(2n+1))).
/// The convergence of these partial products to pi/2 is the classical
/// consequence of the asymptotics of the integrals integral_0^pi Real.sin^n,
/// which is beyond the current library; that statement is recorded as a
/// comment at the end of the file.
from nat import Nat, from_nat, alt_induction, distrib_left, mul_one_right, suc_sub_one, add_suc_right, add_assoc, pow_distrib_mul, one_pow
from real.real_field import Real, mul_div, real_no_zero_divisors
from real.real_ring import mul_pos_pos
from real.real_base import pos_gt_zero, gt_zero_imp_pos
from real.finite_product_mean import finite_real_product, finite_real_product_suc,
    finite_real_product_zero, from_nat_real_pos_of_ne_zero
from real.am_gm import positive_on, finite_real_product_pos, div_pos_of_pos_pos
from real.taylor_general import real_pow_two_eq_mul
from real.pi import pi_over_two
from real.real_seq import converges_to

numerals Nat
numerals Real

/// The (k+1)-th positive even integer, 2k + 2.
define even_term(k: Nat) -> Real {
    from_nat[Real](Nat.2 * k + Nat.2)
}

/// The (k+1)-th positive odd integer, 2k + 1.
define odd_term(k: Nat) -> Real {
    from_nat[Real](Nat.2 * k + Nat.1)
}

/// The (k+1)-th odd integer starting at three, 2k + 3.
define odd_shifted_term(k: Nat) -> Real {
    from_nat[Real](Nat.2 * k + Nat.3)
}

/// The k-th factor of Wallis' product, for k at least one:
/// (2k/(2k-1)) * (2k/(2k+1)).
define wallis_factor(k: Nat) -> Real {
    (from_nat[Real](Nat.2 * k) / from_nat[Real](Nat.2 * k - Nat.1)) *
    (from_nat[Real](Nat.2 * k) / from_nat[Real](Nat.2 * k + Nat.1))
}

/// The Wallis factor at index k+1: the first n factors of Wallis' product are
/// wallis_term(0), ..., wallis_term(n-1), i.e. the factors for k = 1..n.
/// Written out, wallis_term(k) = (2(k+1)/(2k+1)) * (2(k+1)/(2k+3)).
define wallis_term(k: Nat) -> Real {
    (from_nat[Real](Nat.2 * k + Nat.2) / from_nat[Real](Nat.2 * k + Nat.1)) *
    (from_nat[Real](Nat.2 * k + Nat.2) / from_nat[Real](Nat.2 * k + Nat.3))
}

/// The product 2*4*...*2n of the first n even positive integers.
define even_prod(n: Nat) -> Real {
    finite_real_product(even_term, n)
}

/// The product 1*3*...*(2n-1) of the first n odd positive integers.
define odd_prod(n: Nat) -> Real {
    finite_real_product(odd_term, n)
}

/// The product 3*5*...*(2n+1).
define odd_shifted_prod(n: Nat) -> Real {
    finite_real_product(odd_shifted_term, n)
}

/// The n-th partial product of Wallis' product,
/// prod_{k=1}^n (2k/(2k-1)) * (2k/(2k+1)).
define wallis_partial(n: Nat) -> Real {
    finite_real_product(wallis_term, n)
}

/// The defining evaluation of the even terms.
theorem even_term_eq(k: Nat) {
    even_term(k) = from_nat[Real](Nat.2 * k + Nat.2)
} by {
}

/// The defining evaluation of the odd terms.
theorem odd_term_eq(k: Nat) {
    odd_term(k) = from_nat[Real](Nat.2 * k + Nat.1)
} by {
}

/// The defining evaluation of the shifted odd terms.
theorem odd_shifted_term_eq(k: Nat) {
    odd_shifted_term(k) = from_nat[Real](Nat.2 * k + Nat.3)
} by {
}

/// The even terms are positive.
theorem even_term_pos(k: Nat) {
    even_term(k) > Real.0
} by {
    even_term_eq(k)
    from_nat_real_pos_of_ne_zero(Nat.2 * k + Nat.2)
    from_nat[Real](Nat.2 * k + Nat.2) > Real.0
    even_term(k) > Real.0
}

/// The odd terms are positive.
theorem odd_term_pos(k: Nat) {
    odd_term(k) > Real.0
} by {
    odd_term_eq(k)
    from_nat_real_pos_of_ne_zero(Nat.2 * k + Nat.1)
    from_nat[Real](Nat.2 * k + Nat.1) > Real.0
    odd_term(k) > Real.0
}

/// The shifted odd terms are positive.
theorem odd_shifted_term_pos(k: Nat) {
    odd_shifted_term(k) > Real.0
} by {
    odd_shifted_term_eq(k)
    from_nat_real_pos_of_ne_zero(Nat.2 * k + Nat.3)
    from_nat[Real](Nat.2 * k + Nat.3) > Real.0
    odd_shifted_term(k) > Real.0
}

/// The odd denominators 2k + 1 of Wallis' factors are nonzero.
theorem wallis_odd_den_ne_zero(k: Nat) {
    from_nat[Real](Nat.2 * k + Nat.1) != Real.0
} by {
    odd_term_pos(k)
    odd_term_eq(k)
    from_nat[Real](Nat.2 * k + Nat.1) > Real.0
    from_nat[Real](Nat.2 * k + Nat.1) != Real.0
}

/// The shifted odd denominators 2k + 3 of Wallis' factors are nonzero.
theorem wallis_shifted_den_ne_zero(k: Nat) {
    from_nat[Real](Nat.2 * k + Nat.3) != Real.0
} by {
    odd_shifted_term_pos(k)
    odd_shifted_term_eq(k)
    from_nat[Real](Nat.2 * k + Nat.3) > Real.0
    from_nat[Real](Nat.2 * k + Nat.3) != Real.0
}

/// The Wallis factor at index k+1, in explicit form.
theorem wallis_term_eq(k: Nat) {
    wallis_term(k) = (from_nat[Real](Nat.2 * k + Nat.2) / from_nat[Real](Nat.2 * k + Nat.1)) *
        (from_nat[Real](Nat.2 * k + Nat.2) / from_nat[Real](Nat.2 * k + Nat.3))
} by {
}

/// Doubling a successor: 2*(k+1) = 2k + 2.
theorem two_mul_suc_add(k: Nat) {
    Nat.2 * (k + Nat.1) = Nat.2 * k + Nat.2
} by {
    distrib_left(Nat.2, k, Nat.1)
    Nat.2 * (k + Nat.1) = Nat.2 * k + Nat.2 * Nat.1
    mul_one_right(Nat.2)
    Nat.2 * Nat.1 = Nat.2
    Nat.2 * k + Nat.2 * Nat.1 = Nat.2 * k + Nat.2
    Nat.2 * (k + Nat.1) = Nat.2 * k + Nat.2
}

/// The shifted index of the Wallis factor: 2*(k+1) - 1 = 2k + 1.
theorem two_mul_suc_sub_one(k: Nat) {
    Nat.2 * (k + Nat.1) - Nat.1 = Nat.2 * k + Nat.1
} by {
    two_mul_suc_add(k)
    suc_sub_one(Nat.2 * k + Nat.1)
    (Nat.2 * k + Nat.1).suc - Nat.1 = Nat.2 * k + Nat.1
    add_suc_right(Nat.2 * k, Nat.1)
    Nat.2 * k + Nat.1.suc = (Nat.2 * k + Nat.1).suc
    Nat.1.suc = Nat.2
    Nat.2 * k + Nat.2 = (Nat.2 * k + Nat.1).suc
    Nat.2 * (k + Nat.1) - Nat.1 = Nat.2 * k + Nat.1
}

/// The shifted index of the Wallis factor: 2*(k+1) + 1 = 2k + 3.
theorem two_mul_suc_add_one(k: Nat) {
    Nat.2 * (k + Nat.1) + Nat.1 = Nat.2 * k + Nat.3
} by {
    two_mul_suc_add(k)
    add_assoc(Nat.2 * k, Nat.2, Nat.1)
    Nat.2 * k + Nat.2 + Nat.1 = Nat.2 * k + (Nat.2 + Nat.1)
    Nat.2 + Nat.1 = Nat.3
    Nat.2 * k + (Nat.2 + Nat.1) = Nat.2 * k + Nat.3
    Nat.2 * k + Nat.2 + Nat.1 = Nat.2 * k + Nat.3
    Nat.2 * (k + Nat.1) + Nat.1 = Nat.2 * k + Nat.3
}

/// The Wallis factor at index k+1 is the k-th Wallis factor of the classic
/// form (2k/(2k-1)) * (2k/(2k+1)).
theorem wallis_term_factor(k: Nat) {
    wallis_term(k) = wallis_factor(k + Nat.1)
} by {
    wallis_term_eq(k)
    two_mul_suc_add(k)
    two_mul_suc_sub_one(k)
    two_mul_suc_add_one(k)
    wallis_factor(k + Nat.1) = (from_nat[Real](Nat.2 * (k + Nat.1)) /
        from_nat[Real](Nat.2 * (k + Nat.1) - Nat.1)) *
        (from_nat[Real](Nat.2 * (k + Nat.1)) /
        from_nat[Real](Nat.2 * (k + Nat.1) + Nat.1))
    wallis_term(k) = (from_nat[Real](Nat.2 * (k + Nat.1)) /
        from_nat[Real](Nat.2 * (k + Nat.1) - Nat.1)) *
        (from_nat[Real](Nat.2 * (k + Nat.1)) /
        from_nat[Real](Nat.2 * (k + Nat.1) + Nat.1))
    wallis_term(k) = wallis_factor(k + Nat.1)
}

/// Splitting the last factor off the n-th Wallis partial product.
theorem wallis_partial_suc(k: Nat) {
    wallis_partial(k.suc) = wallis_partial(k) * wallis_term(k)
} by {
    finite_real_product_suc(wallis_term, k)
    wallis_partial(k.suc) = finite_real_product(wallis_term, k) * wallis_term(k)
    wallis_partial(k) = finite_real_product(wallis_term, k)
    wallis_partial(k.suc) = wallis_partial(k) * wallis_term(k)
}

/// Splitting the last factor off the even product.
theorem even_prod_suc(k: Nat) {
    even_prod(k.suc) = even_prod(k) * even_term(k)
} by {
    finite_real_product_suc(even_term, k)
    even_prod(k.suc) = finite_real_product(even_term, k) * even_term(k)
    even_prod(k) = finite_real_product(even_term, k)
    even_prod(k.suc) = even_prod(k) * even_term(k)
}

/// Splitting the last factor off the odd product.
theorem odd_prod_suc(k: Nat) {
    odd_prod(k.suc) = odd_prod(k) * odd_term(k)
} by {
    finite_real_product_suc(odd_term, k)
    odd_prod(k.suc) = finite_real_product(odd_term, k) * odd_term(k)
    odd_prod(k) = finite_real_product(odd_term, k)
    odd_prod(k.suc) = odd_prod(k) * odd_term(k)
}

/// Splitting the last factor off the shifted odd product.
theorem odd_shifted_prod_suc(k: Nat) {
    odd_shifted_prod(k.suc) = odd_shifted_prod(k) * odd_shifted_term(k)
} by {
    finite_real_product_suc(odd_shifted_term, k)
    odd_shifted_prod(k.suc) = finite_real_product(odd_shifted_term, k) * odd_shifted_term(k)
    odd_shifted_prod(k) = finite_real_product(odd_shifted_term, k)
    odd_shifted_prod(k.suc) = odd_shifted_prod(k) * odd_shifted_term(k)
}

/// The empty even product is one.
theorem even_prod_zero {
    even_prod(Nat.0) = Real.1
} by {
    finite_real_product_zero(even_term)
    even_prod(Nat.0) = Real.1
}

/// The empty odd product is one.
theorem odd_prod_zero {
    odd_prod(Nat.0) = Real.1
} by {
    finite_real_product_zero(odd_term)
    odd_prod(Nat.0) = Real.1
}

/// The empty shifted odd product is one.
theorem odd_shifted_prod_zero {
    odd_shifted_prod(Nat.0) = Real.1
} by {
    finite_real_product_zero(odd_shifted_term)
    odd_shifted_prod(Nat.0) = Real.1
}

/// A purely algebraic step for the Wallis telescoping: multiplying the
/// quotient of squares by the two Wallis fractions regroups into the quotient
/// of the successor products, provided all denominators are nonzero.
theorem wallis_field_step(e: Real, o: Real, s: Real, a: Real, b: Real, c: Real) {
    o != Real.0 and s != Real.0 and b != Real.0 and c != Real.0 implies
    (e * a).pow(Nat.2) / ((o * b) * (s * c)) =
    (e.pow(Nat.2) / (o * s)) * (a / b) * (a / c)
} by {
    if o != Real.0 and s != Real.0 and b != Real.0 and c != Real.0 {
        pow_distrib_mul[Real](e, a, Nat.2)
        (e * a).pow(Nat.2) = e.pow(Nat.2) * a.pow(Nat.2)
        real_pow_two_eq_mul(e)
        e.pow(Nat.2) = e * e
        real_pow_two_eq_mul(a)
        a.pow(Nat.2) = a * a
        (e * a).pow(Nat.2) = (e * e) * (a * a)
        mul_div(a, b, a, c)
        (a / b) * (a / c) = (a * a) / (b * c)
        ((e * e) / (o * s)) * (a / b) * (a / c) = ((e * e) / (o * s)) * ((a * a) / (b * c))
        real_no_zero_divisors(o, s)
        o * s != Real.0
        real_no_zero_divisors(b, c)
        b * c != Real.0
        mul_div(e * e, o * s, a * a, b * c)
        ((e * e) / (o * s)) * ((a * a) / (b * c)) = (e * e * a * a) / ((o * s) * (b * c))
        ((e * e) / (o * s)) * (a / b) * (a / c) = (e * e * a * a) / ((o * s) * (b * c))
        (e * a).pow(Nat.2) / ((o * b) * (s * c)) = (e * e * a * a) / ((o * s) * (b * c))
        (e * a).pow(Nat.2) / ((o * b) * (s * c)) = ((e * e) / (o * s)) * (a / b) * (a / c)
    }
}

/// The odd terms are positive on every initial segment.
theorem odd_term_positive_on(n: Nat) {
    positive_on(odd_term, n)
} by {
    forall(i: Nat) {
        if i < n {
            odd_term_pos(i)
            odd_term(i) > Real.0
        }
    }
}

/// The shifted odd terms are positive on every initial segment.
theorem odd_shifted_term_positive_on(n: Nat) {
    positive_on(odd_shifted_term, n)
} by {
    forall(i: Nat) {
        if i < n {
            odd_shifted_term_pos(i)
            odd_shifted_term(i) > Real.0
        }
    }
}

/// The product 1*3*...*(2n-1) is positive.
theorem odd_prod_pos(n: Nat) {
    odd_prod(n) > Real.0
} by {
    odd_term_positive_on(n)
    finite_real_product_pos(odd_term, n)
    positive_on(odd_term, n) implies odd_prod(n) > Real.0
    odd_prod(n) > Real.0
}

/// The product 3*5*...*(2n+1) is positive.
theorem odd_shifted_prod_pos(n: Nat) {
    odd_shifted_prod(n) > Real.0
} by {
    odd_shifted_term_positive_on(n)
    finite_real_product_pos(odd_shifted_term, n)
    positive_on(odd_shifted_term, n) implies odd_shifted_prod(n) > Real.0
    odd_shifted_prod(n) > Real.0
}

/// The product 1*3*...*(2n-1) is nonzero.
theorem odd_prod_ne_zero(n: Nat) {
    odd_prod(n) != Real.0
} by {
    odd_prod_pos(n)
    odd_prod(n) > Real.0
    odd_prod(n) != Real.0
}

/// The product 3*5*...*(2n+1) is nonzero.
theorem odd_shifted_prod_ne_zero(n: Nat) {
    odd_shifted_prod(n) != Real.0
} by {
    odd_shifted_prod_pos(n)
    odd_shifted_prod(n) > Real.0
    odd_shifted_prod(n) != Real.0
}

/// The Wallis telescoping step: if the k-th partial product equals the quotient
/// of the k-th products, then the same holds for k+1.
theorem wallis_telescoping_step(k: Nat) {
    wallis_partial(k) = even_prod(k).pow(Nat.2) / (odd_prod(k) * odd_shifted_prod(k))
    implies
    wallis_partial(k.suc) = even_prod(k.suc).pow(Nat.2) / (odd_prod(k.suc) * odd_shifted_prod(k.suc))
} by {
    if wallis_partial(k) = even_prod(k).pow(Nat.2) / (odd_prod(k) * odd_shifted_prod(k)) {
        wallis_partial_suc(k)
        wallis_partial(k.suc) = wallis_partial(k) * wallis_term(k)
        wallis_partial(k.suc) = (even_prod(k).pow(Nat.2) / (odd_prod(k) * odd_shifted_prod(k))) * wallis_term(k)
        wallis_odd_den_ne_zero(k)
        from_nat[Real](Nat.2 * k + Nat.1) != Real.0
        wallis_shifted_den_ne_zero(k)
        from_nat[Real](Nat.2 * k + Nat.3) != Real.0
        odd_prod_ne_zero(k)
        odd_prod(k) != Real.0
        odd_shifted_prod_ne_zero(k)
        odd_shifted_prod(k) != Real.0
        wallis_field_step(even_prod(k), odd_prod(k), odd_shifted_prod(k),
            from_nat[Real](Nat.2 * k + Nat.2), from_nat[Real](Nat.2 * k + Nat.1),
            from_nat[Real](Nat.2 * k + Nat.3))
        (even_prod(k) * from_nat[Real](Nat.2 * k + Nat.2)).pow(Nat.2) /
            ((odd_prod(k) * from_nat[Real](Nat.2 * k + Nat.1)) *
            (odd_shifted_prod(k) * from_nat[Real](Nat.2 * k + Nat.3))) =
            (even_prod(k).pow(Nat.2) / (odd_prod(k) * odd_shifted_prod(k))) *
            (from_nat[Real](Nat.2 * k + Nat.2) / from_nat[Real](Nat.2 * k + Nat.1)) *
            (from_nat[Real](Nat.2 * k + Nat.2) / from_nat[Real](Nat.2 * k + Nat.3))
        wallis_term_eq(k)
        wallis_term(k) = (from_nat[Real](Nat.2 * k + Nat.2) / from_nat[Real](Nat.2 * k + Nat.1)) *
            (from_nat[Real](Nat.2 * k + Nat.2) / from_nat[Real](Nat.2 * k + Nat.3))
        (even_prod(k) * from_nat[Real](Nat.2 * k + Nat.2)).pow(Nat.2) /
            ((odd_prod(k) * from_nat[Real](Nat.2 * k + Nat.1)) *
            (odd_shifted_prod(k) * from_nat[Real](Nat.2 * k + Nat.3))) =
            (even_prod(k).pow(Nat.2) / (odd_prod(k) * odd_shifted_prod(k))) * wallis_term(k)
        even_prod_suc(k)
        even_prod(k.suc) = even_prod(k) * even_term(k)
        even_term_eq(k)
        even_term(k) = from_nat[Real](Nat.2 * k + Nat.2)
        even_prod(k.suc) = even_prod(k) * from_nat[Real](Nat.2 * k + Nat.2)
        odd_prod_suc(k)
        odd_prod(k.suc) = odd_prod(k) * odd_term(k)
        odd_term_eq(k)
        odd_term(k) = from_nat[Real](Nat.2 * k + Nat.1)
        odd_prod(k.suc) = odd_prod(k) * from_nat[Real](Nat.2 * k + Nat.1)
        odd_shifted_prod_suc(k)
        odd_shifted_prod(k.suc) = odd_shifted_prod(k) * odd_shifted_term(k)
        odd_shifted_term_eq(k)
        odd_shifted_term(k) = from_nat[Real](Nat.2 * k + Nat.3)
        odd_shifted_prod(k.suc) = odd_shifted_prod(k) * from_nat[Real](Nat.2 * k + Nat.3)
        even_prod(k.suc).pow(Nat.2) / (odd_prod(k.suc) * odd_shifted_prod(k.suc)) =
            (even_prod(k).pow(Nat.2) / (odd_prod(k) * odd_shifted_prod(k))) * wallis_term(k)
        wallis_partial(k.suc) = even_prod(k.suc).pow(Nat.2) / (odd_prod(k.suc) * odd_shifted_prod(k.suc))
    }
}

/// Wallis' partial products telescope: the first n factors of Wallis' product
/// equal (2*4*...*2n)^2 / ((1*3*...*(2n-1)) * (3*5*...*(2n+1))).
theorem wallis_partial_telescoping(n: Nat) {
    wallis_partial(n) = even_prod(n).pow(Nat.2) / (odd_prod(n) * odd_shifted_prod(n))
} by {
    define p(k: Nat) -> Bool {
        wallis_partial(k) = even_prod(k).pow(Nat.2) / (odd_prod(k) * odd_shifted_prod(k))
    }
    finite_real_product_zero(wallis_term)
    wallis_partial(Nat.0) = Real.1
    even_prod_zero
    even_prod(Nat.0) = Real.1
    odd_prod_zero
    odd_prod(Nat.0) = Real.1
    odd_shifted_prod_zero
    odd_shifted_prod(Nat.0) = Real.1
    one_pow[Real](Nat.2)
    Real.1.pow(Nat.2) = Real.1
    even_prod(Nat.0).pow(Nat.2) = Real.1
    even_prod(Nat.0).pow(Nat.2) / (odd_prod(Nat.0) * odd_shifted_prod(Nat.0)) = Real.1
    wallis_partial(Nat.0) = even_prod(Nat.0).pow(Nat.2) / (odd_prod(Nat.0) * odd_shifted_prod(Nat.0))
    p(Nat.0)
    forall(k: Nat) {
        if p(k) {
            wallis_telescoping_step(k)
            p(k) = (wallis_partial(k) = even_prod(k).pow(Nat.2) / (odd_prod(k) * odd_shifted_prod(k)))
            wallis_partial(k) = even_prod(k).pow(Nat.2) / (odd_prod(k) * odd_shifted_prod(k))
            wallis_partial(k.suc) = even_prod(k.suc).pow(Nat.2) / (odd_prod(k.suc) * odd_shifted_prod(k.suc))
            p(k.suc) = (wallis_partial(k.suc) = even_prod(k.suc).pow(Nat.2) / (odd_prod(k.suc) * odd_shifted_prod(k.suc)))
            p(k.suc)
        }
    }
    p(Nat.0) and forall(k: Nat) { p(k) implies p(k.suc) }
    alt_induction(p)
    forall(k: Nat) { p(k) }
    p(n)
}

/// The k-th Wallis factor is positive.
theorem wallis_term_pos(k: Nat) {
    wallis_term(k) > Real.0
} by {
    wallis_term_eq(k)
    even_term_eq(k)
    from_nat[Real](Nat.2 * k + Nat.2) > Real.0
    odd_term_eq(k)
    from_nat[Real](Nat.2 * k + Nat.1) > Real.0
    odd_shifted_term_eq(k)
    from_nat[Real](Nat.2 * k + Nat.3) > Real.0
    div_pos_of_pos_pos(from_nat[Real](Nat.2 * k + Nat.2), from_nat[Real](Nat.2 * k + Nat.1))
    from_nat[Real](Nat.2 * k + Nat.2) / from_nat[Real](Nat.2 * k + Nat.1) > Real.0
    div_pos_of_pos_pos(from_nat[Real](Nat.2 * k + Nat.2), from_nat[Real](Nat.2 * k + Nat.3))
    from_nat[Real](Nat.2 * k + Nat.2) / from_nat[Real](Nat.2 * k + Nat.3) > Real.0
    gt_zero_imp_pos(from_nat[Real](Nat.2 * k + Nat.2) / from_nat[Real](Nat.2 * k + Nat.1))
    (from_nat[Real](Nat.2 * k + Nat.2) / from_nat[Real](Nat.2 * k + Nat.1)).is_positive
    gt_zero_imp_pos(from_nat[Real](Nat.2 * k + Nat.2) / from_nat[Real](Nat.2 * k + Nat.3))
    (from_nat[Real](Nat.2 * k + Nat.2) / from_nat[Real](Nat.2 * k + Nat.3)).is_positive
    mul_pos_pos(from_nat[Real](Nat.2 * k + Nat.2) / from_nat[Real](Nat.2 * k + Nat.1),
        from_nat[Real](Nat.2 * k + Nat.2) / from_nat[Real](Nat.2 * k + Nat.3))
    ((from_nat[Real](Nat.2 * k + Nat.2) / from_nat[Real](Nat.2 * k + Nat.1)) *
        (from_nat[Real](Nat.2 * k + Nat.2) / from_nat[Real](Nat.2 * k + Nat.3))).is_positive
    pos_gt_zero((from_nat[Real](Nat.2 * k + Nat.2) / from_nat[Real](Nat.2 * k + Nat.1)) *
        (from_nat[Real](Nat.2 * k + Nat.2) / from_nat[Real](Nat.2 * k + Nat.3)))
    (from_nat[Real](Nat.2 * k + Nat.2) / from_nat[Real](Nat.2 * k + Nat.1)) *
        (from_nat[Real](Nat.2 * k + Nat.2) / from_nat[Real](Nat.2 * k + Nat.3)) > Real.0
    wallis_term(k) > Real.0
}

/// The Wallis factors are positive on every initial segment.
theorem wallis_term_positive_on(n: Nat) {
    positive_on(wallis_term, n)
} by {
    forall(i: Nat) {
        if i < n {
            wallis_term_pos(i)
            wallis_term(i) > Real.0
        }
    }
}

/// Every partial product of Wallis' factors is positive.
theorem wallis_partial_pos(n: Nat) {
    wallis_partial(n) > Real.0
} by {
    wallis_term_positive_on(n)
    finite_real_product_pos(wallis_term, n)
    positive_on(wallis_term, n) implies wallis_partial(n) > Real.0
    wallis_partial(n) > Real.0
}

// ---------------------------------------------------------------------------
// The link to pi (stated, not proved here)
// ---------------------------------------------------------------------------

/// The link between the partial products and pi runs through the integrals
/// I_n = integral(x => x.sin.pow(n), Real.0, pi).  Integration by parts
/// gives the reduction I_n = ((n-1)/n) * I_{n-2}, and hence the closed forms
///     I_{2m}   = (pi/2) * (1*3*...*(2m-1)) / (2*4*...*2m),
///     I_{2m+1} = (2*4*...*2m) / (1*3*...*(2m+1)).
/// Since 0 <= Real.sin <= 1 on [0, pi], the ratio I_{2m+1} / I_{2m} tends to one,
/// and dividing the two closed forms gives Wallis' product:
///     pi/2 = prod_{k=1}^infinity (2k/(2k-1)) * (2k/(2k+1)).
/// The library does not yet contain the power-integral reduction (integration
/// by parts for powers of sine) or the monotone-ratio argument needed for the
/// limit, so the full product statement is recorded below, commented out.
/// The theorem wallis_partial_telescoping above gives the closed form of the
/// n-th partial product, and wallis_partial_pos shows the partial products
/// stay positive, so the missing ingredient is only the value pi/2 of the
/// limit.
///
/// // Wallis' product: the limit of the partial products of Wallis' factors
/// // is pi / 2.
/// // theorem wallis_product {
/// //     converges_to(wallis_partial, pi_over_two)
/// // }
///
/// // Equivalently, the partial products converge and their limit is pi / 2.
/// // theorem wallis_product_limit {
/// //     converges(wallis_partial) and limit(wallis_partial) = pi_over_two
/// // }
