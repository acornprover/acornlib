from data.basic.set import Set, compl_contains_eq, double_inclusion, intersection_contains_eq,
    intersection_contains_intro, set_ext, union_contains_left, union_contains_right
from real.real_field import Real
from real.topology import closure, closure_mono, closure_subset_of_closed_set,
    union_closure_subset_closure_union, union_subset_union_closure, is_closed_set,
    is_open_set
from real.topology_closure import closure_is_closed
from real.topology_closed_open import complement_of_open_is_closed
from real.topology_complements import complement_of_closed_is_open
from real.topology_open_intersection import intersection_of_open_is_open

/// The complement of a binary union is the intersection of complements.
theorem complement_union_eq_intersection_complements(s: Set[Real], t: Set[Real]) {
    s.union(t).c = s.c.intersection(t.c)
} by {
    let u = s.union(t).c
    let v = s.c.intersection(t.c)
    forall(x: Real) {
        if u.contains(x) {
            compl_contains_eq(s.union(t), x)
            not s.union(t).contains(x)
            if s.contains(x) {
                union_contains_left(s, t, x)
                false
            }
            if t.contains(x) {
                union_contains_right(s, t, x)
                false
            }
            not s.contains(x)
            not t.contains(x)
            compl_contains_eq(s, x)
            compl_contains_eq(t, x)
            s.c.contains(x)
            t.c.contains(x)
            intersection_contains_intro(s.c, t.c, x)
            v.contains(x)
        }
        if v.contains(x) {
            intersection_contains_eq(s.c, t.c, x)
            s.c.contains(x)
            t.c.contains(x)
            compl_contains_eq(s, x)
            compl_contains_eq(t, x)
            not s.contains(x)
            not t.contains(x)
            not s.union(t).contains(x)
            compl_contains_eq(s.union(t), x)
            u.contains(x)
        }
        u.contains(x) = v.contains(x)
    }
    set_ext(u, v)
}

/// The union of two closed real sets is closed.
theorem union_of_closed_is_closed(s: Set[Real], t: Set[Real]) {
    is_closed_set(s) and is_closed_set(t) implies is_closed_set(s.union(t))
} by {
    if is_closed_set(s) and is_closed_set(t) {
        complement_of_closed_is_open(s)
        complement_of_closed_is_open(t)
        is_open_set(s.c)
        is_open_set(t.c)
        intersection_of_open_is_open(s.c, t.c)
        is_open_set(s.c.intersection(t.c))
        complement_union_eq_intersection_complements(s, t)
        s.union(t).c = s.c.intersection(t.c)
        is_open_set(s.union(t).c)
        complement_of_open_is_closed(s.union(t))
        is_closed_set(s.union(t))
    }
}

/// The closure of a union is contained in the union of closures.
theorem closure_union_subset(s: Set[Real], t: Set[Real]) {
    closure(s.union(t)).subset(closure(s).union(closure(t)))
} by {
    union_subset_union_closure(s, t)
    s.union(t).subset(closure(s).union(closure(t)))
    closure_mono(s.union(t), closure(s).union(closure(t)))
    closure(s.union(t)).subset(closure(closure(s).union(closure(t))))
    closure_is_closed(s)
    closure_is_closed(t)
    is_closed_set(closure(s))
    is_closed_set(closure(t))
    union_of_closed_is_closed(closure(s), closure(t))
    is_closed_set(closure(s).union(closure(t)))
    closure_subset_of_closed_set(closure(s).union(closure(t)))
    closure(closure(s).union(closure(t))).subset(closure(s).union(closure(t)))
    forall(x: Real) {
        if closure(s.union(t)).contains(x) {
            closure(closure(s).union(closure(t))).contains(x)
            closure(s).union(closure(t)).contains(x)
        }
    }
}

/// Closure distributes over binary union.
theorem closure_union_eq(s: Set[Real], t: Set[Real]) {
    closure(s.union(t)) = closure(s).union(closure(t))
} by {
    union_closure_subset_closure_union(s, t)
    closure_union_subset(s, t)
    closure(s).union(closure(t)).subset(closure(s.union(t)))
    closure(s.union(t)).subset(closure(s).union(closure(t)))
    double_inclusion(closure(s.union(t)), closure(s).union(closure(t)))
}
