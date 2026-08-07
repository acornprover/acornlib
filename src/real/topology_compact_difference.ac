from data.basic.set import Set
from real.real_field import Real
from real.topology import is_bounded_real_set, is_closed_set, is_open_set
from real.topology_bounded_algebra import difference_of_bounded_real_set_is_bounded
from real.topology_compact import compact_real_set_is_bounded, compact_real_set_is_closed,
    is_compact_real_set
from real.topology_difference import difference_of_closed_and_open_is_closed

/// Removing an open real set from a compact real set preserves compactness.
theorem difference_of_compact_and_open_is_compact(s: Set[Real], t: Set[Real]) {
    is_compact_real_set(s) and is_open_set(t) implies is_compact_real_set(s.difference(t))
} by {
    if is_compact_real_set(s) and is_open_set(t) {
        compact_real_set_is_closed(s)
        is_closed_set(s)
        difference_of_closed_and_open_is_closed(s, t)
        is_closed_set(s.difference(t))
        compact_real_set_is_bounded(s)
        is_bounded_real_set(s)
        difference_of_bounded_real_set_is_bounded(s, t)
        is_bounded_real_set(s.difference(t))
        is_closed_set(s.difference(t)) and is_bounded_real_set(s.difference(t))
        is_compact_real_set(s.difference(t))
    }
}
