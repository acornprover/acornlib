/// Limit theorems for real sequences: the fundamental limit laws.
///
/// The sequence-level limit algebra (sum, product, quotient) lives in
/// real_seq.ac, prod_seq.ac, limit_algebra.ac, and lhopital.ac; this file
/// restates the headline theorems in a single place, together with the
/// squeeze theorem and the fundamental limit 1/n -> 0.
from nat import Nat, from_nat
from rat import Rat, iop, iop_ne_zero, iop_recip
from real.real_base import Real, add_from_rat
from real.real_seq import converges, converges_to, limit, add_seq, lift_seq,
    converges_to_unique, converges_to_imp_converges, converges_imp_converges_to,
    iop_limit
from real.real_ring import from_nat_is_from_rat
from real.real_series import const_seq, seq_lte, neg_seq, mul_seq, const_converges,
    const_limit, const_converges_to, neg_seq_converges, limit_neg_seq
from real.prod_seq import prod_seq
from real.limits import same_limit, squeeze_theorem
from real.limit_algebra import converges_to_add_seq, converges_to_prod_seq
from real.harmonic import harmonic
from real.integral_trig import sin_le_one, sin_ge_neg_one
from real.uniform_continuity import div_le_div_pos
from real.from_rat_field_hom import real_from_rat_inverse
from real.lhopital import seq_pointwise_eq_converges_to

// ---------------------------------------------------------------------------
// Helpers: extracting facts from the same-limit relation.
// ---------------------------------------------------------------------------

/// If two sequences have the same limit, the first converges.
theorem same_limit_left_converges(a: Nat -> Real, b: Nat -> Real) {
    same_limit(a, b) implies converges(a)
} by {
    if same_limit(a, b) {
        same_limit(a, b) = (converges(a) and converges(b) and limit(a) = limit(b))
        converges(a)
    }
}

/// If two sequences have the same limit, the second converges.
theorem same_limit_right_converges(a: Nat -> Real, b: Nat -> Real) {
    same_limit(a, b) implies converges(b)
} by {
    if same_limit(a, b) {
        same_limit(a, b) = (converges(a) and converges(b) and limit(a) = limit(b))
        converges(b)
    }
}

/// If two sequences have the same limit, their limits are equal.
theorem same_limit_limits_equal(a: Nat -> Real, b: Nat -> Real) {
    same_limit(a, b) implies limit(a) = limit(b)
} by {
    if same_limit(a, b) {
        same_limit(a, b) = (converges(a) and converges(b) and limit(a) = limit(b))
        limit(a) = limit(b)
    }
}

// ---------------------------------------------------------------------------
// The limit of a constant sequence is that constant.
// ---------------------------------------------------------------------------

/// A constant sequence converges.
theorem const_seq_converges(c: Real) {
    converges(const_seq(c))
} by {
    const_converges(c)
    converges(const_seq(c))
}

/// A constant sequence converges to that constant.
theorem const_seq_converges_to(c: Real) {
    converges_to(const_seq(c), c)
} by {
    const_converges_to(c)
    converges_to(const_seq(c), c)
}

/// The limit of a constant sequence is that constant.
theorem const_seq_has_limit(c: Real) {
    limit(const_seq(c)) = c
} by {
    const_limit(c)
    limit(const_seq(c)) = c
}

// ---------------------------------------------------------------------------
// The limit of a sum is the sum of the limits.
// ---------------------------------------------------------------------------

/// If a_n -> l and b_n -> m, then a_n + b_n -> l + m.
theorem limit_of_sum(a: Nat -> Real, b: Nat -> Real, l: Real, m: Real) {
    converges_to(a, l) and converges_to(b, m)
    implies converges_to(add_seq(a, b), l + m)
} by {
    converges_to_add_seq(a, b, l, m)
    converges_to(add_seq(a, b), l + m)
}

// ---------------------------------------------------------------------------
// The limit of a product is the product of the limits.
// ---------------------------------------------------------------------------

/// If a_n -> l and b_n -> m, then a_n * b_n -> l * m.
theorem limit_of_product(a: Nat -> Real, b: Nat -> Real, l: Real, m: Real) {
    converges_to(a, l) and converges_to(b, m)
    implies converges_to(prod_seq(a, b), l * m)
} by {
    converges_to_prod_seq(a, b, l, m)
    converges_to(prod_seq(a, b), l * m)
}

// ---------------------------------------------------------------------------
// The fundamental limit 1/n -> 0.
// ---------------------------------------------------------------------------

/// The sequence n -> 1/(n+1) converges to zero.
///
/// This is the fundamental limit 1/n -> 0, indexed from zero; it is proved by
/// identifying the sequence with the lifted rational sequence iop(n) =
/// 1/(1+n), whose convergence to zero is already in the library.
theorem one_over_suc_converges_to_zero {
    converges_to(harmonic, Real.0)
} by {
    iop_limit
    converges_to(lift_seq(iop), Real.0)
    forall(n: Nat) {
        lift_seq(iop, n) = Real.from_rat(iop(n))
        iop(n) != Rat.0
        iop_recip(n)
        iop(n).inverse = Rat.1 + Rat.from_nat(n)
        real_from_rat_inverse(iop(n))
        Real.from_rat(iop(n).inverse) = Real.from_rat(iop(n)).inverse
        Real.from_rat(iop(n)).inverse = Real.from_rat(Rat.1 + Rat.from_nat(n))
        add_from_rat(Rat.1, Rat.from_nat(n))
        Real.from_rat(Rat.1) + Real.from_rat(Rat.from_nat(n)) =
            Real.from_rat(Rat.1 + Rat.from_nat(n))
        from_nat_is_from_rat(n)
        Real.from_rat(Rat.from_nat(n)) = from_nat[Real](n)
        Real.1 = Real.from_rat(Rat.1)
        Real.from_rat(Rat.1) + Real.from_rat(Rat.from_nat(n)) = Real.1 + from_nat[Real](n)
        from_nat[Real](n.suc) = from_nat[Real](n) + Real.1
        Real.from_rat(Rat.1 + Rat.from_nat(n)) = from_nat[Real](n.suc)
        Real.from_rat(iop(n)).inverse = from_nat[Real](n.suc)
        Real.from_rat(iop(n)) = from_nat[Real](n.suc).inverse
        from_nat[Real](n.suc).inverse = Real.1 / from_nat[Real](n.suc)
        harmonic(n) = Real.1 / from_nat[Real](n.suc)
        lift_seq(iop, n) = harmonic(n)
    }
    seq_pointwise_eq_converges_to(lift_seq(iop), harmonic, Real.0)
    converges_to(harmonic, Real.0)
}

/// The sequence n -> 1/(n+1) converges.
theorem one_over_suc_converges {
    converges(harmonic)
} by {
    one_over_suc_converges_to_zero
    converges_to_imp_converges(harmonic, Real.0)
    converges(harmonic)
}

/// The limit of the sequence n -> 1/(n+1) is zero.
theorem limit_one_over_suc {
    limit(harmonic) = Real.0
} by {
    one_over_suc_converges_to_zero
    converges_to(harmonic, Real.0)
    converges(harmonic)
    converges_imp_converges_to(harmonic)
    converges_to(harmonic, limit(harmonic))
    converges_to_unique(harmonic, Real.0, limit(harmonic))
    Real.0 = limit(harmonic)
    limit(harmonic) = Real.0
}

// ---------------------------------------------------------------------------
// The squeeze theorem.
// ---------------------------------------------------------------------------

/// The squeeze theorem: if a_n <= b_n <= c_n and both a_n and c_n converge to
/// l, then b_n converges to l.
theorem squeeze_converges_to(a: Nat -> Real, b: Nat -> Real, c: Nat -> Real, l: Real) {
    seq_lte(a, b) and seq_lte(b, c) and converges_to(a, l) and converges_to(c, l)
    implies converges_to(b, l)
} by {
    if seq_lte(a, b) and seq_lte(b, c) and converges_to(a, l) and converges_to(c, l) {
        converges_to_imp_converges(a, l)
        converges(a)
        converges_to_imp_converges(c, l)
        converges(c)
        converges_imp_converges_to(a)
        converges_to(a, limit(a))
        converges_to_unique(a, l, limit(a))
        l = limit(a)
        converges_imp_converges_to(c)
        converges_to(c, limit(c))
        converges_to_unique(c, l, limit(c))
        l = limit(c)
        limit(a) = l
        limit(c) = l
        limit(a) = limit(c)
        same_limit(a, c)
        squeeze_theorem(a, b, c)
        same_limit(a, b)
        same_limit_right_converges(a, b)
        converges(b)
        same_limit_limits_equal(a, b)
        limit(a) = limit(b)
        limit(b) = l
        converges_imp_converges_to(b)
        converges_to(b, limit(b))
        converges_to(b, l)
    }
}

// ---------------------------------------------------------------------------
// Concrete squeeze: (Real.sin n)/n -> 0 via -1/n <= (Real.sin n)/n <= 1/n.
// ---------------------------------------------------------------------------

/// The sequence n -> (n+1).sin/(n+1).
define sin_over_suc(n: Nat) -> Real {
    (from_nat[Real](n.suc)).sin / from_nat[Real](n.suc)
}

/// The sequence n -> -1/(n+1), the lower bound in the squeeze.
define neg_one_over_suc(n: Nat) -> Real {
    -(Real.1 / from_nat[Real](n.suc))
}

/// The lower bound -1/(n+1) is termwise at most (n+1).sin/(n+1).
theorem neg_one_over_suc_lte_sin_over_suc {
    seq_lte(neg_one_over_suc, sin_over_suc)
} by {
    forall(n: Nat) {
        let t = from_nat[Real](n.suc)
        t > Real.0
        sin_ge_neg_one(t)
        -Real.1 <= t.sin
        div_le_div_pos(-Real.1, t.sin, t)
        -Real.1 / t <= t.sin / t
        sin_over_suc(n) = t.sin / t
        neg_one_over_suc(n) = -(Real.1 / t)
        -(Real.1 / t) = -Real.1 / t
        neg_one_over_suc(n) <= sin_over_suc(n)
    }
}

/// The upper bound (n+1).sin/(n+1) is termwise at most 1/(n+1).
theorem sin_over_suc_lte_one_over_suc {
    seq_lte(sin_over_suc, harmonic)
} by {
    forall(n: Nat) {
        let t = from_nat[Real](n.suc)
        t > Real.0
        sin_le_one(t)
        t.sin <= Real.1
        div_le_div_pos(t.sin, Real.1, t)
        t.sin / t <= Real.1 / t
        sin_over_suc(n) = t.sin / t
        harmonic(n) = Real.1 / t
        sin_over_suc(n) <= harmonic(n)
    }
}

/// The lower bound -1/(n+1) and the upper bound 1/(n+1) have the same limit.
theorem neg_one_over_suc_same_limit_one_over_suc {
    same_limit(neg_one_over_suc, harmonic)
} by {
    one_over_suc_converges
    converges(harmonic)
    neg_seq_converges(harmonic)
    forall(n: Nat) {
        neg_seq(harmonic, n) = mul_seq(-Real.1, harmonic, n)
        mul_seq(-Real.1, harmonic, n) = -Real.1 * harmonic(n)
        harmonic(n) = Real.1 / from_nat[Real](n.suc)
        -Real.1 * (Real.1 / from_nat[Real](n.suc)) = -(Real.1 / from_nat[Real](n.suc))
        neg_one_over_suc(n) = -(Real.1 / from_nat[Real](n.suc))
        neg_seq(harmonic, n) = neg_one_over_suc(n)
    }
    neg_one_over_suc = neg_seq(harmonic)
    converges(neg_one_over_suc)
    limit_neg_seq(harmonic)
    limit(neg_seq(harmonic)) = -limit(harmonic)
    limit(harmonic) = Real.0
    -limit(harmonic) = -Real.0
    -Real.0 = Real.0
    limit(neg_seq(harmonic)) = Real.0
    neg_one_over_suc = neg_seq(harmonic)
    limit(neg_one_over_suc) = Real.0
    limit(neg_one_over_suc) = limit(harmonic)
    same_limit(neg_one_over_suc, harmonic)
}

/// Squeeze: the sequence (Real.sin n)/n converges to zero.
theorem sin_over_suc_converges_to_zero {
    converges_to(sin_over_suc, Real.0)
} by {
    neg_one_over_suc_lte_sin_over_suc
    seq_lte(neg_one_over_suc, sin_over_suc)
    sin_over_suc_lte_one_over_suc
    seq_lte(sin_over_suc, harmonic)
    neg_one_over_suc_same_limit_one_over_suc
    same_limit(neg_one_over_suc, harmonic)
    squeeze_theorem(neg_one_over_suc, sin_over_suc, harmonic)
    same_limit(neg_one_over_suc, sin_over_suc)
    same_limit_right_converges(neg_one_over_suc, sin_over_suc)
    converges(sin_over_suc)
    same_limit_limits_equal(neg_one_over_suc, sin_over_suc)
    limit(neg_one_over_suc) = limit(sin_over_suc)
    limit(neg_one_over_suc) = Real.0
    limit(sin_over_suc) = Real.0
    converges_imp_converges_to(sin_over_suc)
    converges_to(sin_over_suc, limit(sin_over_suc))
    converges_to(sin_over_suc, Real.0)
}
