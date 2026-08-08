from algebra.mul import Mul
from nat import Nat
from int import Int
from rat import Rat
from real.real_seq import Real, limit_rat, mul_rat_seq, rat_seq, eq_seq, converges, lift_seq, converges_to, limit, add_rat_seq, const_rat_seq, zero_rat_seq, eventual_lb

/// The product of two real numbers.
instance Real: Mul {
    define mul(self, other: Real) -> Real {
        limit_rat(mul_rat_seq(rat_seq(self), rat_seq(other)))
    }
}

theorem real_mul_comm(x: Real, y: Real) {
    x * y = y * x
} by {
    forall(n: Nat) {
        mul_rat_seq(rat_seq(x), rat_seq(y), n) = rat_seq(x, n) * rat_seq(y, n)
        mul_rat_seq(rat_seq(y), rat_seq(x), n) = rat_seq(y, n) * rat_seq(x, n)
        rat_seq(x, n) * rat_seq(y, n) = rat_seq(y, n) * rat_seq(x, n)
        mul_rat_seq(rat_seq(x), rat_seq(y), n) = mul_rat_seq(rat_seq(y), rat_seq(x), n)
    }
    mul_rat_seq(rat_seq(x), rat_seq(y)) = mul_rat_seq(rat_seq(y), rat_seq(x))
}

theorem mul3_as_seq(x: Real, y: Real, z: Real) {
    x * y * z = limit_rat(mul_rat_seq(mul_rat_seq(rat_seq(x), rat_seq(y)), rat_seq(z)))
} by {
    eq_seq(mul_rat_seq(rat_seq(x * y), rat_seq(z)), mul_rat_seq(mul_rat_seq(rat_seq(x), rat_seq(y)), rat_seq(z)))
    limit_rat(mul_rat_seq(rat_seq(x * y), rat_seq(z))) = x * y * z
    converges_to(lift_seq(mul_rat_seq(mul_rat_seq(rat_seq(x), rat_seq(y)), rat_seq(z))), limit_rat(mul_rat_seq(rat_seq(x * y), rat_seq(z))))
    converges(lift_seq(mul_rat_seq(mul_rat_seq(rat_seq(x), rat_seq(y)), rat_seq(z))))
    converges_to(lift_seq(mul_rat_seq(mul_rat_seq(rat_seq(x), rat_seq(y)), rat_seq(z))), x * y * z)
    limit(lift_seq(mul_rat_seq(mul_rat_seq(rat_seq(x), rat_seq(y)), rat_seq(z)))) = x * y * z
}

theorem mul_seq_assoc(a: Nat -> Rat, b: Nat -> Rat, c: Nat -> Rat) {
    mul_rat_seq(mul_rat_seq(a, b), c) = mul_rat_seq(a, mul_rat_seq(b, c))
} by {
    forall(n: Nat) {
        a(n) * b(n) = mul_rat_seq(a, b, n)
        b(n) * c(n) = mul_rat_seq(b, c, n)
        a(n) * mul_rat_seq(b, c, n) = mul_rat_seq(a, mul_rat_seq(b, c), n)
        mul_rat_seq(a, b, n) * c(n) = mul_rat_seq(mul_rat_seq(a, b), c, n)
        a(n) * (b(n) * c(n)) = a(n) * b(n) * c(n)
        mul_rat_seq(mul_rat_seq(a, b), c, n) = mul_rat_seq(a, mul_rat_seq(b, c), n)
    }
}

theorem mul_assoc(x: Real, y: Real, z: Real) {
    x * (y * z) = (x * y) * z
} by {
    mul_rat_seq(mul_rat_seq(rat_seq(x), rat_seq(y)), rat_seq(z)) = mul_rat_seq(rat_seq(x), mul_rat_seq(rat_seq(y), rat_seq(z)))
    mul_rat_seq(rat_seq(x), mul_rat_seq(rat_seq(y), rat_seq(z))) = mul_rat_seq(mul_rat_seq(rat_seq(y), rat_seq(z)), rat_seq(x))
}

theorem limit_rat_rat_seq(x: Real) {
    x = limit_rat(rat_seq(x))
}

theorem limit_definition_of_add(x: Real, y: Real) {
    x + y = limit_rat(add_rat_seq(rat_seq(x), rat_seq(y)))
} by {
    converges_to(lift_seq(add_rat_seq(rat_seq(x), rat_seq(y))), x + y)
}

theorem eq_seq_rat_seq_limit(a: Nat -> Rat) {
    converges(lift_seq(a)) implies
    eq_seq(a, rat_seq(limit_rat(a)))
}

theorem eq_seq_imp_limit_rat_eq(a: Nat -> Rat, b: Nat -> Rat) {
    eq_seq(a, b) implies
    limit_rat(a) = limit_rat(b)
} by {
    if eq_seq(a, b) {
        eq_seq(b, a)
        eq_seq(b, b)
        converges_to(lift_seq(b), limit_rat(a))
        converges_to(lift_seq(b), limit_rat(b))
        limit_rat(a) = limit_rat(b)
    }
}

theorem limit_rat_add(a: Nat -> Rat, b: Nat -> Rat) {
    converges(lift_seq(a)) and converges(lift_seq(b))
    implies
    limit_rat(add_rat_seq(a, b)) = limit_rat(a) + limit_rat(b)
} by {
    if converges(lift_seq(a)) and converges(lift_seq(b)) {
        converges_to(lift_seq(add_rat_seq(a, b)), limit_rat(a) + limit_rat(b))
        converges(lift_seq(rat_seq(limit_rat(a) + limit_rat(b))))
        converges_to(lift_seq(add_rat_seq(a, b)), limit_rat(rat_seq(limit_rat(a) + limit_rat(b))))
        eq_seq(rat_seq(limit_rat(a) + limit_rat(b)), add_rat_seq(a, b))
        limit_rat(rat_seq(limit_rat(a) + limit_rat(b))) = limit_rat(add_rat_seq(a, b))
        limit_rat(rat_seq(limit_rat(a) + limit_rat(b))) = limit_rat(a) + limit_rat(b)
        limit_rat(add_rat_seq(a, b)) = limit_rat(a) + limit_rat(b)
    }
}

theorem mul_distrib_right(x: Real, y: Real, z: Real) {
    x * (y + z) = x * y + x * z
} by {
    // Reduce the lhs to sequence form.
    let syz: Nat -> Rat = add_rat_seq(rat_seq(y), rat_seq(z))
    converges(lift_seq(rat_seq(y)))
    converges(lift_seq(rat_seq(z)))
    converges_to(lift_seq(add_rat_seq(rat_seq(y), rat_seq(z))), limit_rat(rat_seq(y)) + limit_rat(rat_seq(z)))
    converges(lift_seq(add_rat_seq(rat_seq(y), rat_seq(z))))
    limit_rat(add_rat_seq(rat_seq(y), rat_seq(z))) = y + z
    converges_to(lift_seq(rat_seq(limit_rat(add_rat_seq(rat_seq(y), rat_seq(z))))), limit_rat(add_rat_seq(rat_seq(y), rat_seq(z))))
    eq_seq(add_rat_seq(rat_seq(y), rat_seq(z)), rat_seq(limit_rat(add_rat_seq(rat_seq(y), rat_seq(z)))))
    eq_seq(add_rat_seq(rat_seq(y), rat_seq(z)), rat_seq(y + z))
    eq_seq(syz, rat_seq(y + z))
    converges(lift_seq(rat_seq(x)))
    eq_seq(mul_rat_seq(rat_seq(x), syz), mul_rat_seq(rat_seq(x), rat_seq(y + z)))
    limit_rat(mul_rat_seq(rat_seq(x), rat_seq(y + z))) = limit_rat(mul_rat_seq(rat_seq(x), syz))
    limit_rat(mul_rat_seq(rat_seq(x), rat_seq(y + z))) = x * (y + z)
    x * (y + z) = limit_rat(mul_rat_seq(rat_seq(x), add_rat_seq(rat_seq(y), rat_seq(z))))

    // Reduce the rhs to sequence form.
    let s1: Nat -> Rat = mul_rat_seq(rat_seq(x), rat_seq(y))
    let s2: Nat -> Rat = mul_rat_seq(rat_seq(x), rat_seq(z))
    converges(lift_seq(s2))
    limit_rat(s1) + limit_rat(s2) = limit_rat(add_rat_seq(s1, s2))

    // Now observe that the sequences themselves are equal.
    forall(n: Nat) {
        mul_rat_seq(rat_seq(x), add_rat_seq(rat_seq(y), rat_seq(z)), n) = rat_seq(x, n) * add_rat_seq(rat_seq(y), rat_seq(z), n)
        add_rat_seq(rat_seq(y), rat_seq(z), n) = rat_seq(y, n) + rat_seq(z, n)
        rat_seq(x, n) * add_rat_seq(rat_seq(y), rat_seq(z), n) = rat_seq(x, n) * (rat_seq(y, n) + rat_seq(z, n))
        rat_seq(x, n) * (rat_seq(y, n) + rat_seq(z, n)) = rat_seq(x, n) * rat_seq(y, n) + rat_seq(x, n) * rat_seq(z, n)
        rat_seq(x, n) * rat_seq(y, n) = mul_rat_seq(rat_seq(x), rat_seq(y), n)
        rat_seq(x, n) * rat_seq(z, n) = mul_rat_seq(rat_seq(x), rat_seq(z), n)
        mul_rat_seq(rat_seq(x), rat_seq(y), n) + mul_rat_seq(rat_seq(x), rat_seq(z), n) = add_rat_seq(mul_rat_seq(rat_seq(x), rat_seq(y)), mul_rat_seq(rat_seq(x), rat_seq(z)), n)
        mul_rat_seq(rat_seq(x), add_rat_seq(rat_seq(y), rat_seq(z)), n) = add_rat_seq(mul_rat_seq(rat_seq(x), rat_seq(y)), mul_rat_seq(rat_seq(x), rat_seq(z)), n)
    }
    mul_rat_seq(rat_seq(x), add_rat_seq(rat_seq(y), rat_seq(z))) = add_rat_seq(mul_rat_seq(rat_seq(x), rat_seq(y)), mul_rat_seq(rat_seq(x), rat_seq(z)))
}

theorem mul_distrib_left(x: Real, y: Real, z: Real) {
    (x + y) * z = x * z + y * z
}

theorem limit_rat_mul_rat_seq(a: Nat -> Rat, b: Nat -> Rat) {
    converges(lift_seq(a)) and converges(lift_seq(b))
    implies
    limit_rat(mul_rat_seq(a, b)) = limit_rat(a) * limit_rat(b)
} by {
    if converges(lift_seq(a)) and converges(lift_seq(b)) {
        eq_seq(a, rat_seq(limit_rat(a)))
        eq_seq(b, rat_seq(limit_rat(b)))
        eq_seq(mul_rat_seq(a, b), mul_rat_seq(rat_seq(limit_rat(a)), rat_seq(limit_rat(b))))
        limit_rat(mul_rat_seq(rat_seq(limit_rat(a)), rat_seq(limit_rat(b)))) = limit_rat(mul_rat_seq(a, b))
        limit_rat(mul_rat_seq(rat_seq(limit_rat(a)), rat_seq(limit_rat(b)))) = limit_rat(a) * limit_rat(b)
        limit_rat(mul_rat_seq(a, b)) = limit_rat(a) * limit_rat(b)
    }
}

theorem limit_rat_const_rat_seq(r: Rat) {
    limit_rat(const_rat_seq(r)) = Real.from_rat(r)
} by {
    converges(lift_seq(rat_seq(Real.from_rat(r))))
    converges_to(lift_seq(const_rat_seq(r)), Real.from_rat(r))
    limit_rat(rat_seq(Real.from_rat(r))) = Real.from_rat(r)
    eq_seq(rat_seq(Real.from_rat(r)), const_rat_seq(r))
    limit_rat(rat_seq(Real.from_rat(r))) = limit_rat(const_rat_seq(r))
}

theorem mul_one_right(x: Real) {
    x * Real.1 = x
} by {
    converges(lift_seq(rat_seq(x)))
    converges_to(lift_seq(const_rat_seq(Rat.1)), Real.from_rat(Rat.1))
    converges(lift_seq(const_rat_seq(Rat.1)))
    limit_rat(rat_seq(x)) * limit_rat(const_rat_seq(Rat.1)) = limit_rat(mul_rat_seq(rat_seq(x), const_rat_seq(Rat.1)))
    forall(n: Nat) {
        mul_rat_seq(rat_seq(x), const_rat_seq(Rat.1), n) = rat_seq(x, n) * const_rat_seq(Rat.1, n)
        const_rat_seq(Rat.1, n) = Rat.1
        rat_seq(x, n) * Rat.1 = rat_seq(x, n)
        mul_rat_seq(rat_seq(x), const_rat_seq(Rat.1), n) = rat_seq(x, n)
    }
    mul_rat_seq(rat_seq(x), const_rat_seq(Rat.1)) = rat_seq(x)
}

theorem mul_one_left(x: Real) {
    Real.1 * x = x
}

theorem mul_zero_right(x: Real) {
    x * Real.0 = Real.0
} by {
    converges(lift_seq(rat_seq(x)))
    converges_to(lift_seq(zero_rat_seq), Real.0)
    eq_seq(zero_rat_seq, zero_rat_seq)
    converges(lift_seq(zero_rat_seq))
    limit_rat(rat_seq(x)) * limit_rat(zero_rat_seq) = limit_rat(mul_rat_seq(rat_seq(x), zero_rat_seq))
    forall(n: Nat) {
        mul_rat_seq(rat_seq(x), zero_rat_seq, n) = rat_seq(x, n) * zero_rat_seq(n)
        zero_rat_seq(n) = const_rat_seq(Rat.0, n)
        const_rat_seq(Rat.0, n) = Rat.0
        zero_rat_seq(n) = Rat.0
        rat_seq(x, n) * Rat.0 = Rat.0
        mul_rat_seq(rat_seq(x), zero_rat_seq, n) = zero_rat_seq(n)
    }
    mul_rat_seq(rat_seq(x), zero_rat_seq) = zero_rat_seq
}

theorem mul_zero_left(x: Real) {
    Real.0 * x = Real.0
}

theorem mul_neg_one_left(x: Real) {
    -Real.1 * x = -x
} by {
    x + -Real.1 * x = Real.0
}

theorem mul_neg_one_right(x: Real) {
    x * -Real.1 = -x
}

theorem mul_neg_left(x: Real, y: Real) {
    -x * y = -(x * y)
}

theorem mul_neg_right(x: Real, y: Real) {
    x * -y = -(x * y)
}

theorem mul_sub_distrib_right(x: Real, y: Real, z: Real) {
    x * (y - z) = x * y - x * z
}

theorem mul_sub_distrib_left(x: Real, y: Real, z: Real) {
    (x - y) * z = x * z - y * z
}

theorem real_lte_imp_rat_lte(p: Rat, q: Rat) {
    Real.from_rat(p) <= Real.from_rat(q) implies
    p <= q
}

theorem pos_lte_imp_pos(x: Real, y: Real) {
    x.is_positive and x <= y implies
    y.is_positive
}

theorem mul_pos_pos(x: Real, y: Real) {
    x.is_positive and y.is_positive
    implies
    (x * y).is_positive
} by {
    let x_lb: Rat satisfy {
        x_lb.is_positive and Real.from_rat(x_lb) < x
    }
    let nx: Nat satisfy {
        forall(i: Nat) {
            nx <= i implies
            Real.from_rat(x_lb) <= lift_seq(rat_seq(x))(i)
        }
    }
    let y_lb: Rat satisfy {
        y_lb.is_positive and Real.from_rat(y_lb) < y
    }
    let ny: Nat satisfy {
        forall(i: Nat) {
            ny <= i implies
            Real.from_rat(y_lb) <= lift_seq(rat_seq(y))(i)
        }
    }
    let n: Nat satisfy {
        nx <= n and ny <= n
    }
    forall(i: Nat) {
        if n <= i {
            Real.from_rat(x_lb) <= lift_seq(rat_seq(x))(i)
            Real.from_rat(y_lb) <= lift_seq(rat_seq(y))(i)
            x_lb * y_lb <= mul_rat_seq(rat_seq(x), rat_seq(y))(i)
            Real.from_rat(x_lb * y_lb) <= lift_seq(mul_rat_seq(rat_seq(x), rat_seq(y)))(i)
        }
    }
    forall(i: Nat) {
        n <= i implies Real.from_rat(x_lb * y_lb) <= lift_seq(mul_rat_seq(rat_seq(x), rat_seq(y)))(i)
    }
    exists(n0: Nat) {
        forall(i: Nat) {
            n0 <= i implies
            Real.from_rat(x_lb * y_lb) <= lift_seq(mul_rat_seq(rat_seq(x), rat_seq(y)))(i)
        }
    }
    eventual_lb(lift_seq(mul_rat_seq(rat_seq(x), rat_seq(y))), Real.from_rat(x_lb * y_lb))
    Real.from_rat(x_lb * y_lb) <= limit_rat(mul_rat_seq(rat_seq(x), rat_seq(y)))
}

theorem mul_neg_pos(x: Real, y: Real) {
    x.is_negative and y.is_positive
    implies
    (x * y).is_negative
} by {
    --x = x
    (-x).is_positive
    (-x * y).is_positive
    --x * y = -(-x * y)
    (-(-x * y)).is_negative
}

theorem mul_pos_neg(x: Real, y: Real) {
    x.is_positive and y.is_negative
    implies
    (x * y).is_negative
}

theorem mul_neg_neg(x: Real, y: Real) {
    x.is_negative and y.is_negative
    implies
    (x * y).is_positive
}

theorem mul_nonneg_nonneg(x: Real, y: Real) {
    not x.is_negative and not y.is_negative
    implies
    not (x * y).is_negative
} by {
    if x = Real.0 {
    } else {
        if y = Real.0 {
            not (x * y).is_negative
        } else {
            not (x * y).is_negative
        }
    }
}

/// Product of nonnegative numbers is nonnegative.
theorem mul_nonneg(a: Real, b: Real) {
    a >= Real.0 and b >= Real.0 implies a * b >= Real.0
} by {
    if a >= Real.0 and b >= Real.0 {
        not a.is_negative
        not b.is_negative
        not (a * b).is_negative
        not a * b < Real.0
    }
}

theorem square_nonneg(x: Real) {
    x * x >= Real.0
} by {
    x * x < Real.0 or x * x >= Real.0
    x * x <= Real.0 or Real.0 <= x * x
    if x = Real.0 {
    } else {
        if x.is_positive {
        } else {
            x.is_negative
            (x * x).is_positive
            not (x * x).is_negative
            not x * x < Real.0
        }
    }
}

theorem lt_mul_pos_right(x: Real, y: Real, z: Real) {
    z.is_positive and x < y implies
    x * z < y * z
}

theorem lt_mul_pos_left(x: Real, y: Real, z: Real) {
    z.is_positive and x < y implies
    z * x < z * y
}

theorem lte_mul_nonneg_right(x: Real, y: Real, z: Real) {
    not z.is_negative and x <= y implies
    x * z <= y * z
} by {
    x * z <= y * z or x * z > y * z
    x * z <= x * z
    if z = Real.0 {
    } else {
        if x = y {
        } else {
            x * z < y * z
        }
    }
}

theorem lte_mul_nonneg_left(x: Real, y: Real, z: Real) {
    not z.is_negative and x <= y implies
    z * x <= z * y
}

/// Multiplication is monotone for nonnegative reals.
theorem mul_le_mul_nonneg(a: Real, b: Real, c: Real, d: Real) {
    a >= Real.0 and b >= Real.0 and c >= Real.0 and d >= Real.0 and a <= c and b <= d
    implies
    a * b <= c * d
} by {
    a * b <= c * b
    c * b <= c * d
}

theorem lt_mul_neg_right(x: Real, y: Real, z: Real) {
    z.is_negative and x < y implies
    y * z < x * z
} by {
    if z.is_negative and x < y {
        (x - y).is_negative
        ((x - y) * z).is_positive
        x * z - y * z = (x - y) * z
        (x * z - y * z).is_positive
    }
}

theorem lt_mul_neg_left(x: Real, y: Real, z: Real) {
    z.is_negative and x < y implies
    z * y < z * x
}

theorem add_neg_neg(x: Real, y: Real) {
    x.is_negative and y.is_negative implies (x + y).is_negative
} by {
    if x.is_negative and y.is_negative {
        x < Real.0
        y < Real.0
        x + y < Real.0 + Real.0
        Real.0 + Real.0 = Real.0
        x + y < Real.0
    }
}

theorem zero_lte_imp_non_neg(x: Real) {
    Real.0 <= x implies not x.is_negative
}

theorem non_neg_imp_zero_lte(x: Real) {
    not x.is_negative implies Real.0 <= x
}

theorem mul_from_rat(a: Rat, b: Rat) {
    Real.from_rat(a) * Real.from_rat(b) = Real.from_rat(a * b)
} by {
    // Represent the lhs as a limit
    let mr: Nat -> Rat = mul_rat_seq(rat_seq(Real.from_rat(a)), rat_seq(Real.from_rat(b)))

    // Simplify the lhs
    eq_seq(rat_seq(Real.from_rat(a)), const_rat_seq(a))
    eq_seq(rat_seq(Real.from_rat(b)), const_rat_seq(b))
    forall(n: Nat) {
        mul_rat_seq(const_rat_seq(a), const_rat_seq(b), n) = const_rat_seq(a, n) * const_rat_seq(b, n)
        const_rat_seq(a, n) = a
        const_rat_seq(b, n) = b
        const_rat_seq(a * b, n) = a * b
        mul_rat_seq(const_rat_seq(a), const_rat_seq(b), n) = const_rat_seq(a * b, n)
    }
    mul_rat_seq(const_rat_seq(a), const_rat_seq(b)) = const_rat_seq(a * b)
}

theorem gt_pos_is_pos(a: Real, b: Real) {
    a.is_positive and a < b implies b.is_positive
}

theorem exists_small_mul(a: Real, b: Real) {
    not a.is_negative and b.is_positive implies
    exists(c: Real) {
        c.is_positive and a * c < b
    }
} by {
    let a_ub: Rat satisfy {
        a < Real.from_rat(a_ub)
    }
    Real.from_rat(a_ub).is_positive
    a_ub.is_positive
    let b_lb: Rat satisfy {
        b_lb.is_positive and Real.from_rat(b_lb) < b
    }
    let r: Rat satisfy {
        a_ub * r = b_lb
    }
    let c: Real = Real.from_rat(r)
    c.is_positive
    c * a < c * Real.from_rat(a_ub)
    c * a = a * c
    c * Real.from_rat(a_ub) = Real.from_rat(a_ub) * c
    Real.from_rat(a_ub) * Real.from_rat(r) = Real.from_rat(a_ub * r)
    a * c < Real.from_rat(b_lb)
    a * c < b
}

theorem exists_small_mul_variant(a: Real, b: Real) {
    b.is_positive implies
    exists(c: Real) {
        c.is_positive and a.abs * c < b
    }
}

theorem exists_small_mul_variant_2(a: Real, b: Real) {
    a.is_positive and b.is_positive implies
    exists(c: Real) {
        c.is_positive and c * a < b
    }
} by {
    from real.real_base import pos_imp_eq_abs
    if a.is_positive and b.is_positive {
        let c: Real satisfy {
            c.is_positive and a.abs * c < b
        }
        a = a.abs
        c * a = a.abs * c
        c.is_positive and c * a < b
        exists(d: Real) {
            d.is_positive and d * a < b
        }
    }
}

theorem mul_abs(a: Real, b: Real) {
    a.abs * b.abs = (a * b).abs
} by {
    not a.is_negative or a.abs = -a
    not b.is_negative or b.abs = -b
    not (a * b).is_positive or (a * b).abs = a * b
    -a * -b = -(-a * b)
    -a * b = -(a * b)
    a * -b = -(a * b)
    --(a * b) = a * b
    if a.is_negative {
        a.abs = -a
        if b.is_negative {
            b.abs = -b
            (a * b).is_positive
            (a * b).abs = a * b
        } else {
            if b = Real.0 {
                a.abs * b.abs = (a * b).abs
            } else {
                b.is_positive
                (a * b).is_negative
                (a * b).abs = -(a * b)
                a.abs * b.abs = (a * b).abs
            }
        }
    } else {
        if a = Real.0 {
            a.abs * b.abs = (a * b).abs
        } else {
            a.is_positive
            a.abs = a
            if b.is_negative {
                b.abs = -b
                (a * b).is_negative
                (a * b).abs = -(a * b)
                a.abs * b.abs = (a * b).abs
            } else {
                b.abs = b
                not (a * b).is_negative
                a.abs * b.abs = (a * b).abs
            }
        }
    }
}

theorem square_zero_imp_zero(a: Real) {
    a * a = Real.0 implies a = Real.0
}

// Multiplicative algebraic structure.

from algebra.semigroup import Semigroup

instance Real: Semigroup

from algebra.monoid.monoid import Monoid

instance Real: Monoid

from semiring import Semiring

instance Real: Semiring

from algebra.ring.ring import Ring

instance Real: Ring

from algebra.comm_semigroup import CommSemigroup

instance Real: CommSemigroup

from algebra.comm_monoid import CommMonoid

instance Real: CommMonoid

from comm_ring import CommRing

instance Real: CommRing

// Relationship between from_nat and from_rat

from nat import from_nat

/// Rat.from_nat preserves addition.
theorem rat_from_nat_add(m: Nat, n: Nat) {
    Rat.from_nat(m + n) = Rat.from_nat(m) + Rat.from_nat(n)
} by {
    // Rat.from_nat(x) = Rat.from_int(Int.from_nat(x))
    // Int.from_nat preserves addition
    Rat.from_nat(m + n) = Rat.from_int(Int.from_nat(m + n))
    Int.from_nat(m + n) = Int.from_nat(m) + Int.from_nat(n)
    Rat.from_int(Int.from_nat(m) + Int.from_nat(n)) = Rat.from_int(Int.from_nat(m)) + Rat.from_int(Int.from_nat(n))
}

/// The embedding of natural numbers into reals via from_nat equals the composition
/// of Rat.from_nat and Real.from_rat.
theorem from_nat_is_from_rat(n: Nat) {
    from_nat[Real](n) = Real.from_rat(Rat.from_nat(n))
} by {
    define f(k: Nat) -> Bool {
        from_nat[Real](k) = Real.from_rat(Rat.from_nat(k))
    }

    // Base case: k = 0
    from_nat[Real](Nat.0) = Real.0
    Rat.from_nat(Nat.0) = Rat.0
    Real.from_rat(Rat.0) = Real.0
    f(Nat.0)

    // Inductive step
    forall(k: Nat) {
        if f(k) {
            // IH: from_nat[Real](k) = Real.from_rat(Rat.from_nat(k))

            // LHS: from_nat[Real](k.suc)
            from_nat[Real](k.suc) = from_nat[Real](k) + Real.1
            from_nat[Real](k.suc) = Real.from_rat(Rat.from_nat(k)) + Real.1

            // Real.1 = Real.from_rat(Rat.1)
            Real.1 = Real.from_rat(Rat.1)
            from_nat[Real](k.suc) = Real.from_rat(Rat.from_nat(k)) + Real.from_rat(Rat.1)

            // Use add_from_rat
            Real.from_rat(Rat.from_nat(k)) + Real.from_rat(Rat.1) = Real.from_rat(Rat.from_nat(k) + Rat.1)
            from_nat[Real](k.suc) = Real.from_rat(Rat.from_nat(k) + Rat.1)

            // Show Rat.from_nat(k.suc) = Rat.from_nat(k) + Rat.1
            Rat.from_nat(k.suc) = Rat.from_nat(k + Nat.1)
            Rat.from_nat(k + Nat.1) = Rat.from_nat(k) + Rat.from_nat(Nat.1)
            Rat.from_nat(Nat.1) = Rat.1
            Rat.from_nat(k.suc) = Rat.from_nat(k) + Rat.1

            from_nat[Real](k.suc) = Real.from_rat(Rat.from_nat(k.suc))
            f(k.suc)
        }
    }

    f(n)
}
