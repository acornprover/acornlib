from data.basic.set import Set
from real.real_field import Real
from real.topology import adherent_point_eps, bounded_real_set_lower_bound,
    bounded_real_set_upper_bound, closure, closure_contains_eq, is_adherent_point_of_set,
    is_bounded_real_set, is_eps_adherent_to_set

numerals Real

/// The closure of a bounded real set is bounded.
theorem closure_of_bounded_real_set_is_bounded(s: Set[Real]) {
    is_bounded_real_set(s) implies is_bounded_real_set(closure(s))
} by {
    if is_bounded_real_set(s) {
        let bound: Real satisfy {
            bound > Real.0 and forall(z: Real) {
                s.contains(z) implies -bound <= z and z <= bound
            }
        }
        let bound2 = bound + Real.1
        bound2 > Real.0
        forall(x: Real) {
            if closure(s).contains(x) {
                closure_contains_eq(s, x)
                is_adherent_point_of_set(s, x)
                Real.1.is_positive
                adherent_point_eps(s, x, Real.1)
                is_eps_adherent_to_set(s, x, Real.1)
                let y: Real satisfy {
                    s.contains(y) and y.is_close(x, Real.1)
                }
                bounded_real_set_lower_bound(s, bound, y)
                bounded_real_set_upper_bound(s, bound, y)
                -bound <= y
                y <= bound
                y.is_close(x, Real.1)
                (y - x).abs < Real.1
                y - x < Real.1
                -(y - x) < Real.1
                x - y < Real.1
                x < y + Real.1
                y + Real.1 <= bound + Real.1
                x < bound2
                x <= bound2
                x > y - Real.1
                y + -Real.1 >= -bound + -Real.1
                y - Real.1 = y + -Real.1
                -bound - Real.1 = -bound + -Real.1
                y - Real.1 >= -bound - Real.1
                x > -bound - Real.1
                -(bound + Real.1) = -bound - Real.1
                x > -bound2
                -bound2 <= x
                -bound2 <= x and x <= bound2
            }
        }
        exists(bound3: Real) {
            bound3 > Real.0 and forall(x: Real) {
                closure(s).contains(x) implies -bound3 <= x and x <= bound3
            }
        }
    }
}
