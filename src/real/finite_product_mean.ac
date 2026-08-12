from nat import Nat, from_nat, zero_or_suc, lt_and_lte, alt_induction
from list import partial, partial_split_last
from real.real_field import Real, mul_div_cancel
from real.real_ring import mul_nonneg
from real.harmonic import from_nat_suc_pos_real

numerals Nat
numerals Real

/// Product of the first `n` real values of a sequence.
define finite_real_product(f: Nat -> Real, n: Nat) -> Real {
    match n {
        Nat.zero {
            Real.1
        }
        Nat.suc(k) {
            finite_real_product(f, k) * f(k)
        }
    }
}

/// Arithmetic mean of the first `n` real values of a sequence.
/// For `n = 0` this is the field value obtained by division by zero; use the
/// theorems below only with an explicit nonzero-count hypothesis.
define finite_real_mean(f: Nat -> Real, n: Nat) -> Real {
    partial[Real](f, n) / from_nat[Real](n)
}

/// The first `n` values of a real sequence are nonnegative.
define nonnegative_on(f: Nat -> Real, n: Nat) -> Bool {
    forall(i: Nat) { i < n implies f(i) >= Real.0 }
}

/// The finite product over the empty range is one.
theorem finite_real_product_zero(f: Nat -> Real) {
    finite_real_product(f, Nat.0) = Real.1
} by {
    match Nat.0 {
        Nat.zero {
            finite_real_product(f, Nat.0) = Real.1
        }
        Nat.suc(k) {
            false
        }
    }
}

/// Splitting off the last factor of a finite real product.
theorem finite_real_product_suc(f: Nat -> Real, n: Nat) {
    finite_real_product(f, n.suc) = finite_real_product(f, n) * f(n)
} by {
    match n.suc {
        Nat.zero {
            false
        }
        Nat.suc(k) {
            k = n
            finite_real_product(f, n.suc) = finite_real_product(f, n) * f(n)
        }
    }
}

/// The one-term finite product is the single value.
theorem finite_real_product_one(f: Nat -> Real) {
    finite_real_product(f, Nat.1) = f(Nat.0)
} by {
    finite_real_product_suc(f, Nat.0)
    finite_real_product_zero(f)
}


/// Restricted nonnegativity is inherited by shorter initial segments.
theorem nonnegative_on_prefix(f: Nat -> Real, n: Nat, m: Nat) {
    nonnegative_on(f, n) and m <= n implies nonnegative_on(f, m)
} by {
    if nonnegative_on(f, n) and m <= n {
        forall(i: Nat) {
            if i < m {
                lt_and_lte(i, m, n)
                i < n
                nonnegative_on(f, n) = forall(j: Nat) { j < n implies f(j) >= Real.0 }
                f(i) >= Real.0
            }
        }
    }
}

/// A finite product over a nonnegative prefix is nonnegative.
theorem finite_real_product_nonnegative(f: Nat -> Real, n: Nat) {
    nonnegative_on(f, n) implies finite_real_product(f, n) >= Real.0
} by {
    define p(k: Nat) -> Bool {
        nonnegative_on(f, k) implies finite_real_product(f, k) >= Real.0
    }
    finite_real_product_zero(f)
    Real.1 >= Real.0
    p(Nat.0)
    forall(k: Nat) {
        if p(k) {
            if nonnegative_on(f, k.suc) {
                k < k.suc
                k <= k.suc
                nonnegative_on_prefix(f, k.suc, k)
                nonnegative_on(f, k)
                p(k) = (nonnegative_on(f, k) implies finite_real_product(f, k) >= Real.0)
                finite_real_product(f, k) >= Real.0
                nonnegative_on(f, k.suc) = forall(i: Nat) { i < k.suc implies f(i) >= Real.0 }
                f(k) >= Real.0
                mul_nonneg(finite_real_product(f, k), f(k))
                finite_real_product(f, k) * f(k) >= Real.0
                finite_real_product_suc(f, k)
                finite_real_product(f, k.suc) >= Real.0
            }
            p(k.suc)
        }
    }
    p(Nat.0) and forall(k: Nat) { p(k) implies p(k.suc) }
    alt_induction(p)
    forall(k: Nat) { p(k) }
    p(n)
}

/// Natural-to-real counts are positive when the natural count is nonzero.
theorem from_nat_real_pos_of_ne_zero(n: Nat) {
    n != Nat.0 implies from_nat[Real](n) > Real.0
} by {
    if n != Nat.0 {
        zero_or_suc(n)
        let k: Nat satisfy { n = k.suc }
        from_nat_suc_pos_real(k)
        from_nat[Real](n) > Real.0
    }
}

/// Natural-to-real counts are nonzero when the natural count is nonzero.
theorem from_nat_real_ne_zero_of_ne_zero(n: Nat) {
    n != Nat.0 implies from_nat[Real](n) != Real.0
} by {
    if n != Nat.0 {
        from_nat_real_pos_of_ne_zero(n)
        from_nat[Real](n) > Real.0
        from_nat[Real](n) != Real.0
    }
}

/// Multiplying a finite mean by its nonzero count recovers the finite sum.
theorem finite_real_mean_mul_count(f: Nat -> Real, n: Nat) {
    n != Nat.0 implies finite_real_mean(f, n) * from_nat[Real](n) = partial[Real](f, n)
} by {
    if n != Nat.0 {
        from_nat_real_ne_zero_of_ne_zero(n)
        mul_div_cancel(partial[Real](f, n), from_nat[Real](n))
        from_nat[Real](n) * (partial[Real](f, n) / from_nat[Real](n)) = partial[Real](f, n)
        finite_real_mean(f, n) = partial[Real](f, n) / from_nat[Real](n)
        from_nat[Real](n) * finite_real_mean(f, n) = partial[Real](f, n)
        finite_real_mean(f, n) * from_nat[Real](n) = from_nat[Real](n) * finite_real_mean(f, n)
        finite_real_mean(f, n) * from_nat[Real](n) = partial[Real](f, n)
    }
}
