/// Concrete geometric-majorant consumers for real series comparison.
///
/// This module packages common downstream uses of
/// `real.series_comparison_bridge` with explicit geometric majorants. It keeps
/// the accepted bridge's concrete eventual-index API and introduces no
/// additional generic eventual predicate.

from data.basic.functions import function_extensionality
from list import map, partial, sum
from nat import Nat
from real.abs_conv import abs_fn, absolutely_converges, absolutely_converges_imp_converges
from real.real_base import lte_trans_eq
from real.real_ring import Real, mul_nonneg
from real.real_series import comparison_test, converges, geom_converges, is_lower_bound, mul_seq,
    partial_seq_lte, pow_nonneg, seq_lte, tail
from real.series_comparison_bridge import absolutely_converges_iff_tail,
    eventually_abs_dominated_absolutely_converges, eventually_abs_dominated_series_converges,
    eventually_dominated_interval_sum_lte, eventually_dominated_nonneg_series_converges,
    eventually_dominated_tail_interval_sum_lte, series_converges_of_tail,
    tail_lower_bound_of_eventual_nonneg, tail_seq_lte_of_eventual_index_lte

numerals Real

/// The concrete geometric majorant `k ↦ c * r^k`.
define geometric_majorant(c: Real, r: Real, k: Nat) -> Real {
    c * r.pow(k)
}

/// The geometric majorant is the existing scalar-multiple sequence `mul_seq`.
theorem geometric_majorant_eq_mul_seq(c: Real, r: Real) {
    geometric_majorant(c, r) = mul_seq(c, r.pow)
} by {
    forall(k: Nat) {
        mul_seq(c, r.pow, k) = c * r.pow(k)
        geometric_majorant(c, r, k) = mul_seq(c, r.pow, k)
    }
    function_extensionality(geometric_majorant(c, r), mul_seq(c, r.pow))
}

/// Each geometric-majorant term is nonnegative when the coefficient and ratio are nonnegative.
theorem geometric_majorant_nonneg(c: Real, r: Real, k: Nat) {
    Real.0 <= c and Real.0 <= r implies Real.0 <= geometric_majorant(c, r, k)
} by {
    if Real.0 <= c and Real.0 <= r {
        c >= Real.0
        pow_nonneg(r, k)
        Real.0 <= r.pow(k)
        r.pow(k) >= Real.0
        mul_nonneg(c, r.pow(k))
        c * r.pow(k) >= Real.0
        Real.0 <= c * r.pow(k)
        geometric_majorant(c, r, k) = c * r.pow(k)
        c * r.pow(k) = geometric_majorant(c, r, k)
        lte_trans_eq(Real.0, c * r.pow(k), geometric_majorant(c, r, k))
        Real.0 <= geometric_majorant(c, r, k)
    }
}

/// The geometric majorant has zero as a lower bound under nonnegative parameters.
theorem geometric_majorant_lower_bound_zero(c: Real, r: Real) {
    Real.0 <= c and Real.0 <= r implies is_lower_bound(geometric_majorant(c, r), Real.0)
} by {
    if Real.0 <= c and Real.0 <= r {
        forall(k: Nat) {
            geometric_majorant_nonneg(c, r, k)
            Real.0 <= geometric_majorant(c, r, k)
        }
        is_lower_bound(geometric_majorant(c, r), Real.0)
    }
}

/// The partial sums of a geometric majorant are the scalar multiple of geometric partial sums.
theorem partial_geometric_majorant_eq_mul_seq_partial(c: Real, r: Real) {
    partial(geometric_majorant(c, r)) = mul_seq(c, partial(r.pow))
} by {
    geometric_majorant_eq_mul_seq(c, r)
    geometric_majorant(c, r) = mul_seq(c, r.pow)
    partial(geometric_majorant(c, r)) = partial(mul_seq(c, r.pow))
    partial(mul_seq(c, r.pow)) = mul_seq(c, partial(r.pow))
    partial(geometric_majorant(c, r)) = mul_seq(c, partial(r.pow))
}

/// Geometric majorants converge when `0 <= r < 1`.
theorem geometric_majorant_series_converges(c: Real, r: Real) {
    Real.0 <= r and r < Real.1 implies converges(partial(geometric_majorant(c, r)))
} by {
    if Real.0 <= r and r < Real.1 {
        not r.is_negative
        r.abs = r
        r.abs < Real.1
        geom_converges(r)
        converges(partial(r.pow))
        converges(mul_seq(c, partial(r.pow)))
        partial_geometric_majorant_eq_mul_seq_partial(c, r)
        partial(geometric_majorant(c, r)) = mul_seq(c, partial(r.pow))
        converges(partial(geometric_majorant(c, r)))
    }
}

/// Eventual absolute domination by a nonnegative geometric majorant gives absolute convergence.
theorem eventually_abs_le_geometric_absolutely_converges(f: Nat -> Real, c: Real, r: Real, n: Nat) {
    Real.0 <= c
    and Real.0 <= r
    and r < Real.1
    and (forall(k: Nat) { n <= k implies abs_fn(f)(k) <= geometric_majorant(c, r, k) })
    implies absolutely_converges(f)
} by {
    if Real.0 <= c
        and Real.0 <= r
        and r < Real.1
        and (forall(k: Nat) { n <= k implies abs_fn(f)(k) <= geometric_majorant(c, r, k) }) {
        geometric_majorant_series_converges(c, r)
        converges(partial(geometric_majorant(c, r)))
        eventually_abs_dominated_absolutely_converges(f, geometric_majorant(c, r), n)
        absolutely_converges(f)
    }
}

/// Eventual absolute domination by a geometric majorant gives ordinary convergence.
theorem eventually_abs_le_geometric_series_converges(f: Nat -> Real, c: Real, r: Real, n: Nat) {
    Real.0 <= c
    and Real.0 <= r
    and r < Real.1
    and (forall(k: Nat) { n <= k implies abs_fn(f)(k) <= geometric_majorant(c, r, k) })
    implies converges(partial(f))
} by {
    if Real.0 <= c
        and Real.0 <= r
        and r < Real.1
        and (forall(k: Nat) { n <= k implies abs_fn(f)(k) <= geometric_majorant(c, r, k) }) {
        eventually_abs_le_geometric_absolutely_converges(f, c, r, n)
        absolutely_converges(f)
        converges(partial(f))
    }
}

/// Eventual nonnegative domination by a geometric majorant gives ordinary convergence.
theorem eventually_nonneg_le_geometric_series_converges(f: Nat -> Real, c: Real, r: Real, n: Nat) {
    Real.0 <= c
    and Real.0 <= r
    and r < Real.1
    and (forall(k: Nat) { n <= k implies Real.0 <= f(k) })
    and (forall(k: Nat) { n <= k implies f(k) <= geometric_majorant(c, r, k) })
    implies converges(partial(f))
} by {
    if Real.0 <= c
        and Real.0 <= r
        and r < Real.1
        and (forall(k: Nat) { n <= k implies Real.0 <= f(k) })
        and (forall(k: Nat) { n <= k implies f(k) <= geometric_majorant(c, r, k) }) {
        geometric_majorant_series_converges(c, r)
        converges(partial(geometric_majorant(c, r)))
        eventually_dominated_nonneg_series_converges(f, geometric_majorant(c, r), n)
        converges(partial(f))
    }
}

/// Tail-form absolute geometric comparison.
theorem tail_abs_le_geometric_absolutely_converges(f: Nat -> Real, c: Real, r: Real, n: Nat) {
    Real.0 <= c
    and Real.0 <= r
    and r < Real.1
    and (forall(k: Nat) { abs_fn(tail(f, n))(k) <= geometric_majorant(c, r, k) })
    implies absolutely_converges(f)
} by {
    if Real.0 <= c
        and Real.0 <= r
        and r < Real.1
        and (forall(k: Nat) { abs_fn(tail(f, n))(k) <= geometric_majorant(c, r, k) }) {
        forall(k: Nat) {
            Nat.0 <= k
            abs_fn(tail(f, n))(k) <= geometric_majorant(c, r, k)
        }
        forall(k: Nat) { Nat.0 <= k implies abs_fn(tail(f, n))(k) <= geometric_majorant(c, r, k) }
        eventually_abs_le_geometric_absolutely_converges(tail(f, n), c, r, Nat.0)
        absolutely_converges(tail(f, n))
        absolutely_converges_iff_tail(f, n)
        absolutely_converges(f)
    }
}

/// Tail-form nonnegative geometric comparison.
theorem tail_nonneg_le_geometric_series_converges(f: Nat -> Real, c: Real, r: Real, n: Nat) {
    Real.0 <= c
    and Real.0 <= r
    and r < Real.1
    and (forall(k: Nat) { Real.0 <= tail(f, n)(k) })
    and (forall(k: Nat) { tail(f, n)(k) <= geometric_majorant(c, r, k) })
    implies converges(partial(f))
} by {
    if Real.0 <= c
        and Real.0 <= r
        and r < Real.1
        and (forall(k: Nat) { Real.0 <= tail(f, n)(k) })
        and (forall(k: Nat) { tail(f, n)(k) <= geometric_majorant(c, r, k) }) {
        forall(k: Nat) {
            Nat.0 <= k
            Real.0 <= tail(f, n)(k)
            tail(f, n)(k) <= geometric_majorant(c, r, k)
        }
        is_lower_bound(tail(f, n), Real.0)
        seq_lte(tail(f, n), geometric_majorant(c, r))
        geometric_majorant_series_converges(c, r)
        converges(partial(geometric_majorant(c, r)))
        comparison_test(tail(f, n), geometric_majorant(c, r))
        converges(partial(tail(f, n)))
        series_converges_of_tail(f, n)
        converges(partial(f))
    }
}

/// Eventual domination by a geometric majorant passes to tail partial sums.
theorem eventually_geometric_tail_interval_sum_lte(f: Nat -> Real, c: Real, r: Real, n: Nat, k: Nat) {
    (forall(i: Nat) { n <= i implies f(i) <= geometric_majorant(c, r, i) }) implies
        partial(tail(f, n), k) <= partial(tail(geometric_majorant(c, r), n), k)
} by {
    if forall(i: Nat) { n <= i implies f(i) <= geometric_majorant(c, r, i) } {
        eventually_dominated_tail_interval_sum_lte(f, geometric_majorant(c, r), n, k)
        partial(tail(f, n), k) <= partial(tail(geometric_majorant(c, r), n), k)
    }
}

/// Eventual domination by a geometric majorant gives explicit interval-sum bounds.
theorem eventually_geometric_interval_sum_lte(f: Nat -> Real, c: Real, r: Real, n: Nat, k: Nat) {
    (forall(i: Nat) { n <= i implies f(i) <= geometric_majorant(c, r, i) }) implies
        sum(map(n.until(n + k), f)) <= sum(map(n.until(n + k), geometric_majorant(c, r)))
} by {
    if forall(i: Nat) { n <= i implies f(i) <= geometric_majorant(c, r, i) } {
        eventually_dominated_interval_sum_lte(f, geometric_majorant(c, r), n, k)
        sum(map(n.until(n + k), f)) <= sum(map(n.until(n + k), geometric_majorant(c, r)))
    }
}

/// Eventual comparison against a geometric majorant is available on tails.
theorem tail_seq_lte_geometric_of_eventual_lte(f: Nat -> Real, c: Real, r: Real, n: Nat) {
    (forall(i: Nat) { n <= i implies f(i) <= geometric_majorant(c, r, i) }) implies
        seq_lte(tail(f, n), tail(geometric_majorant(c, r), n))
} by {
    if forall(i: Nat) { n <= i implies f(i) <= geometric_majorant(c, r, i) } {
        tail_seq_lte_of_eventual_index_lte(f, geometric_majorant(c, r), n)
        seq_lte(tail(f, n), tail(geometric_majorant(c, r), n))
    }
}

/// A nonnegative eventual hypothesis gives a nonnegative tail for the dominated sequence.
theorem tail_lower_bound_zero_of_eventual_nonneg(f: Nat -> Real, n: Nat) {
    (forall(i: Nat) { n <= i implies Real.0 <= f(i) }) implies is_lower_bound(tail(f, n), Real.0)
} by {
    if forall(i: Nat) { n <= i implies Real.0 <= f(i) } {
        tail_lower_bound_of_eventual_nonneg(f, n)
        is_lower_bound(tail(f, n), Real.0)
    }
}

/// Pointwise geometric domination passes to partial sums.
theorem partial_sums_lte_of_geometric_seq_lte(f: Nat -> Real, c: Real, r: Real) {
    seq_lte(f, geometric_majorant(c, r)) implies seq_lte(partial(f), partial(geometric_majorant(c, r)))
} by {
    partial_seq_lte(f, geometric_majorant(c, r))
}
