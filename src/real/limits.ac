/// Limit comparison utilities for real sequences.
from nat import Nat, lte_trans
from data.basic.functions import compose
from real.real_base import Real
from real.real_series import seq_lte, neg_seq, const_seq
from real.real_seq import converges, converges_to, limit, add_seq, tail_bound, converges_to_unique
from real.prod_seq import prod_seq
from order import is_monotone, monotone_from_forall

/// Two sequences converge and share the same limit.
define same_limit(a: Nat -> Real, b: Nat -> Real) -> Bool {
    converges(a) and converges(b) and limit(a) = limit(b)
}

/// A sequence converges with zero as its limit.
define vanishes(a: Nat -> Real) -> Bool {
    converges(a) and limit(a) = Real.0
}

/// The sequence obtained by taking absolute values termwise.
define abs_seq(a: Nat -> Real, n: Nat) -> Real {
    a(n).abs
}

/// Having the same limit is reflexive.
theorem same_limit_refl(a: Nat -> Real) {
    converges(a) implies same_limit(a, a)
}

/// Having the same limit is symmetric.
theorem same_limit_symm(a: Nat -> Real, b: Nat -> Real) {
    same_limit(a, b) implies same_limit(b, a)
}

/// Having the same limit is transitive.
theorem same_limit_trans(a: Nat -> Real, b: Nat -> Real, c: Nat -> Real) {
    same_limit(a, b) and same_limit(b, c) implies same_limit(a, c)
}

/// Bounding the absolute difference by the difference of arguments.
theorem abs_abs_sub_le_abs_sub(a: Real, b: Real) {
    (a.abs - b.abs).abs <= (a - b).abs
} by {
    let diff = a.abs - b.abs

    ((a - b) + b).abs <= (a - b).abs + b.abs
    a.abs <= (a - b).abs + b.abs
    a.abs + -b.abs <= (a - b).abs + b.abs + -b.abs

    ((b - a) + a).abs <= (b - a).abs + a.abs
    b.abs <= (a - b).abs + a.abs
    b.abs + -a.abs <= (a - b).abs + a.abs + -a.abs

    if diff.is_negative {
        diff.abs = -diff
        -diff = -(a.abs - b.abs)
        -(a.abs - b.abs) = -a.abs + b.abs
        -a.abs + b.abs = b.abs - a.abs
        b.abs - a.abs <= (a - b).abs
        diff.abs <= (a - b).abs
    } else {
    }
}

/// Absolute value applied twice is redundant.
theorem abs_abs(a: Real) {
    a.abs.abs = a.abs
} by {
    if a.is_negative {
    } else {
    }
}

/// Closeness is preserved by taking absolute values.
theorem abs_close_of_close(a: Real, b: Real, eps: Real) {
    a.is_close(b, eps) implies a.abs.is_close(b.abs, eps)
} by {
    (a.abs - b.abs).abs <= (a - b).abs and (a - b).abs < eps implies (a.abs - b.abs).abs < eps
}

/// The absolute value sequence converges to the absolute value of the limit.
theorem limit_abs_seq(a: Nat -> Real) {
    converges(a) implies converges_to(abs_seq(a), limit(a).abs)
} by {
    let lim_abs = limit(a).abs
    converges_to(a, limit(a))
    forall(eps: Real) {
        if eps.is_positive {
            let n: Nat satisfy {
                tail_bound(a, limit(a), n, eps)
            }
            forall(i: Nat) {
                if n <= i {
                    a(i).is_close(limit(a), eps) implies a(i).abs.is_close(limit(a).abs, eps)
                    abs_seq(a)(i).is_close(lim_abs, eps)
                }
            }
            tail_bound(abs_seq(a), lim_abs, n, eps)
        }
    }
}

/// Addition preserves the same-limit relation.
theorem same_limit_add_seq(a: Nat -> Real, b: Nat -> Real, c: Nat -> Real, d: Nat -> Real) {
    same_limit(a, b) and same_limit(c, d)
    implies same_limit(add_seq(a, c), add_seq(b, d))
} by {
    converges(a) and converges(c) implies converges_to(add_seq(a, c), limit(a) + limit(c))
    converges_to(add_seq(a, c), limit(a) + limit(c)) implies converges(add_seq(a, c))

    converges(b) and converges(d) implies converges_to(add_seq(b, d), limit(b) + limit(d))
    converges_to(add_seq(b, d), limit(b) + limit(d)) implies converges(add_seq(b, d))

    converges(add_seq(a, c)) implies converges_to(add_seq(a, c), limit(add_seq(a, c)))
    converges_to(add_seq(a, c), limit(add_seq(a, c))) and converges_to(add_seq(a, c), limit(a) + limit(c)) implies limit(add_seq(a, c)) = limit(a) + limit(c)

    converges(add_seq(b, d)) implies converges_to(add_seq(b, d), limit(add_seq(b, d)))
    converges_to(add_seq(b, d), limit(add_seq(b, d))) and converges_to(add_seq(b, d), limit(b) + limit(d)) implies limit(add_seq(b, d)) = limit(b) + limit(d)

}

/// Multiplication preserves the same-limit relation.
theorem same_limit_prod_seq(a: Nat -> Real, b: Nat -> Real, c: Nat -> Real, d: Nat -> Real) {
    same_limit(a, b) and same_limit(c, d)
    implies same_limit(prod_seq(a, c), prod_seq(b, d))
} by {

    converges(a) and converges(c) implies converges_to(prod_seq(a, c), limit(a) * limit(c))
    converges_to(prod_seq(a, c), limit(a) * limit(c)) implies converges(prod_seq(a, c))

    converges(b) and converges(d) implies converges_to(prod_seq(b, d), limit(b) * limit(d))
    converges_to(prod_seq(b, d), limit(b) * limit(d)) implies converges(prod_seq(b, d))

    converges(prod_seq(a, c)) implies converges_to(prod_seq(a, c), limit(prod_seq(a, c)))
    converges_to(prod_seq(a, c), limit(prod_seq(a, c))) and converges_to(prod_seq(a, c), limit(a) * limit(c)) implies limit(prod_seq(a, c)) = limit(a) * limit(c)

    converges(prod_seq(b, d)) implies converges_to(prod_seq(b, d), limit(prod_seq(b, d)))
    converges_to(prod_seq(b, d), limit(prod_seq(b, d))) and converges_to(prod_seq(b, d), limit(b) * limit(d)) implies limit(prod_seq(b, d)) = limit(b) * limit(d)

}

/// Absolute values preserve the same-limit relation.
theorem same_limit_abs_seq(a: Nat -> Real, b: Nat -> Real) {
    same_limit(a, b) implies same_limit(abs_seq(a), abs_seq(b))
} by {

    converges(a) implies converges_to(abs_seq(a), limit(a).abs)
    converges_to(abs_seq(a), limit(a).abs) implies converges(abs_seq(a))

    converges(b) implies converges_to(abs_seq(b), limit(b).abs)
    converges_to(abs_seq(b), limit(b).abs) implies converges(abs_seq(b))

    converges(abs_seq(a)) implies converges_to(abs_seq(a), limit(abs_seq(a)))
    converges_to(abs_seq(a), limit(abs_seq(a))) and converges_to(abs_seq(a), limit(a).abs) implies limit(abs_seq(a)) = limit(a).abs

    converges(abs_seq(b)) implies converges_to(abs_seq(b), limit(abs_seq(b)))
    converges_to(abs_seq(b), limit(abs_seq(b))) and converges_to(abs_seq(b), limit(b).abs) implies limit(abs_seq(b)) = limit(b).abs

}

/// Vanishing is preserved by taking termwise absolute values.
theorem vanishes_abs_seq(a: Nat -> Real) {
    vanishes(a) implies vanishes(abs_seq(a))
} by {
    if vanishes(a) {
        converges(a)
        limit(a) = Real.0
        limit(a).abs = Real.0
        converges_to(abs_seq(a), limit(a).abs)
        converges_to(abs_seq(a), Real.0)
        converges(abs_seq(a))
        limit(abs_seq(a)) = Real.0
        vanishes(abs_seq(a))
    }
}

/// A sequence dominated by a vanishing absolute value also vanishes.
theorem vanishes_of_abs_lte_abs(a: Nat -> Real, b: Nat -> Real) {
    seq_lte(abs_seq(a), abs_seq(b)) and vanishes(b)
    implies vanishes(a)
} by {

    converges_to(abs_seq(b), Real.0)

    forall(eps: Real) {
        if eps.is_positive {
            let n: Nat satisfy {
                tail_bound(abs_seq(b), Real.0, n, eps)
            }
            forall(i: Nat) {
                if n <= i {
                    abs_seq(a)(i) <= abs_seq(b)(i) and abs_seq(b)(i) < eps implies abs_seq(a)(i) < eps
                    a(i).abs < eps implies a(i).is_close(Real.0, eps)
                    a(i).is_close(Real.0, eps)
                }
            }
            tail_bound(a, Real.0, n, eps)
        }
    }
    converges_to(a, Real.0)
    converges(a)
}

/// Adding a vanishing sequence preserves limits.
theorem same_limit_add_vanish_right(a: Nat -> Real, b: Nat -> Real) {
    converges(a) and vanishes(b)
    implies same_limit(add_seq(a, b), a)
} by {

    converges(a) and converges(b) implies converges_to(add_seq(a, b), limit(a) + limit(b))
    converges_to(add_seq(a, b), limit(a) + limit(b)) implies converges(add_seq(a, b))

    converges_to(add_seq(a, b), limit(add_seq(a, b)))
    converges_to(add_seq(a, b), limit(add_seq(a, b))) and converges_to(add_seq(a, b), limit(a) + limit(b)) implies limit(add_seq(a, b)) = limit(a) + limit(b)

    limit(a) + Real.0 = limit(a)
}

/// Multiplying by a vanishing sequence yields another vanishing sequence.
theorem vanishes_prod_seq(a: Nat -> Real, b: Nat -> Real) {
    converges(a) and vanishes(b)
    implies vanishes(prod_seq(a, b))
} by {

    not vanishes(b) or converges(b)
    not vanishes(b) or Real.0 = limit(b)
    limit(a) * Real.0 = Real.0
    converges(a) and converges(b) implies converges_to(prod_seq(a, b), limit(a) * limit(b))
    not converges(prod_seq(a, b)) or converges_to(prod_seq(a, b), limit(prod_seq(a, b)))
    converges_to(prod_seq(a, b), limit(prod_seq(a, b))) and converges_to(prod_seq(a, b), Real.0) implies limit(prod_seq(a, b)) = Real.0
    limit(prod_seq(a, b)) != Real.0 or not converges(prod_seq(a, b)) or vanishes(prod_seq(a, b))
}

/// If two sequences converge to the same limit, their difference vanishes.
theorem same_limit_imp_vanishes(a: Nat -> Real, b: Nat -> Real) {
    same_limit(a, b) implies vanishes(add_seq(a, neg_seq(b)))
} by {
    if same_limit(a, b) {
        converges(a)
        converges(b)
        let diff = add_seq(a, neg_seq(b))

        converges_to(neg_seq(b), -limit(b))
        converges(neg_seq(b))
        converges_to(neg_seq(b), limit(neg_seq(b)))
        limit(neg_seq(b)) = -limit(b)

        converges_to(diff, limit(a) + limit(neg_seq(b)))

        limit(a) + limit(neg_seq(b)) = limit(a) + -limit(b)

        converges(diff)
        converges_to(diff, limit(diff))
        limit(diff) = limit(a) + limit(neg_seq(b))
        limit(diff) = limit(a) + -limit(b)

        limit(a) = limit(b)
        limit(a) + -limit(b) = limit(a) + -limit(a)
        limit(a) + -limit(a) = Real.0
        limit(diff) = Real.0

        converges(diff)
        vanishes(diff)
        diff = add_seq(a, neg_seq(b))
        vanishes(add_seq(a, neg_seq(b)))
    }
}

/// Squeeze theorem for vanishing: if 0 <= a <= b and b vanishes, then a vanishes.
theorem vanishes_squeeze(a: Nat -> Real, b: Nat -> Real) {
    seq_lte(const_seq(Real.0), a) and seq_lte(a, b) and vanishes(b)
    implies vanishes(a)
} by {
    if seq_lte(const_seq(Real.0), a) and seq_lte(a, b) {
        forall(n: Nat) {
            const_seq(Real.0, n) = Real.0
            Real.0 <= a(n)
            a(n) <= b(n)
            abs_seq(a)(n) = a(n).abs
            abs_seq(b)(n) = b(n).abs
            not a(n).is_negative
            a(n).abs = a(n)
            b(n) <= b(n).abs
            a(n) <= b(n).abs
            a(n).abs <= b(n).abs
            abs_seq(a)(n) <= abs_seq(b)(n)
        }
        seq_lte(abs_seq(a), abs_seq(b))
    }
}

/// Squeeze theorem for convergence: if a <= b <= c and a, c have the same limit, then b has that limit too.
theorem squeeze_theorem(a: Nat -> Real, b: Nat -> Real, c: Nat -> Real) {
    seq_lte(a, b) and seq_lte(b, c) and same_limit(a, c)
    implies same_limit(a, b)
} by {
    if seq_lte(a, b) and seq_lte(b, c) and same_limit(a, c) {
        let diff_ca = add_seq(c, neg_seq(a))
        let diff_ba = add_seq(b, neg_seq(a))

        vanishes(diff_ca)

        forall(n: Nat) {
            diff_ba(n) = add_seq(b, neg_seq(a))(n)
            add_seq(b, neg_seq(a))(n) = b(n) + neg_seq(a)(n)
            neg_seq(a)(n) = -a(n)
            b(n) + -a(n) = b(n) - a(n)
            diff_ba(n) = b(n) - a(n)

            diff_ca(n) = add_seq(c, neg_seq(a))(n)
            add_seq(c, neg_seq(a))(n) = c(n) + neg_seq(a)(n)
            c(n) + -a(n) = c(n) - a(n)
            diff_ca(n) = c(n) - a(n)

            a(n) <= b(n)
            Real.0 <= b(n) - a(n)
            Real.0 <= diff_ba(n)

            b(n) <= c(n)
            b(n) + -a(n) <= c(n) + -a(n)
            b(n) - a(n) <= c(n) - a(n)
            diff_ba(n) <= diff_ca(n)
        }

        forall(n: Nat) {
            diff_ba(n) = add_seq(b, neg_seq(a))(n)
            add_seq(b, neg_seq(a))(n) = b(n) + neg_seq(a)(n)
            neg_seq(a)(n) = -a(n)
            b(n) + -a(n) = b(n) - a(n)
            diff_ba(n) = b(n) - a(n)

            a(n) <= b(n)
            Real.0 <= b(n) - a(n)
            Real.0 <= diff_ba(n)

            const_seq(Real.0)(n) = Real.0
            const_seq(Real.0)(n) <= diff_ba(n)
        }
        seq_lte(const_seq(Real.0), diff_ba)

        forall(n: Nat) {
            diff_ba(n) <= diff_ca(n)
        }
        seq_lte(diff_ba, diff_ca)

        vanishes(diff_ba)

        converges(diff_ba)
        limit(diff_ba) = Real.0

        converges(a)
        limit(a) + limit(diff_ba) = limit(a) + Real.0
        limit(a) + Real.0 = limit(a)

        forall(n: Nat) {
            add_seq(a, diff_ba)(n) = a(n) + diff_ba(n)

            diff_ba(n) = add_seq(b, neg_seq(a))(n)
            add_seq(b, neg_seq(a))(n) = b(n) + neg_seq(a)(n)
            neg_seq(a)(n) = -a(n)
            b(n) + -a(n) = b(n) - a(n)
            diff_ba(n) = b(n) - a(n)

            a(n) + (b(n) - a(n)) = b(n)
            add_seq(a, diff_ba)(n) = b(n)
        }
        add_seq(a, diff_ba) = b

        converges(b)
        limit(b) = limit(a)

        same_limit(a, b)
    }
}

/// True if f: Nat -> Nat tends to infinity (i.e., eventually exceeds any bound).
define tends_to_infinity(f: Nat -> Nat) -> Bool {
    forall(big_n: Nat) {
        exists(m: Nat) {
            forall(n: Nat) {
                m <= n implies big_n <= f(n)
            }
        }
    }
}

/// True if f is nondecreasing.
define is_nondecreasing_nat(f: Nat -> Nat) -> Bool {
    forall(i: Nat, j: Nat) {
        i <= j implies f(i) <= f(j)
    }
}

/// A monotone function on natural numbers is nondecreasing.
theorem monotone_is_nondecreasing_nat(f: Nat -> Nat) {
    is_monotone(f) implies is_nondecreasing_nat(f)
} by {
    if is_monotone(f) {
        forall(i: Nat, j: Nat) {
            if i <= j {
                f(i) <= f(j)
            }
        }
        is_nondecreasing_nat(f)
    }
}

/// True if f is unbounded (i.e., not bounded above by any fixed value).
define is_unbounded(f: Nat -> Nat) -> Bool {
    forall(bound: Nat) {
        exists(n: Nat) {
            bound < f(n)
        }
    }
}

/// A nondecreasing unbounded function tends to infinity.
theorem nondecreasing_unbounded_tends_to_infinity(f: Nat -> Nat) {
    is_nondecreasing_nat(f) and is_unbounded(f)
    implies
    tends_to_infinity(f)
} by {
    forall(big_n: Nat) {
        let m: Nat satisfy {
            big_n < f(m)
        }

        forall(n: Nat) {
            if m <= n {
                f(m) <= f(n)
                big_n < f(m)
                big_n <= f(n)
            }
        }
    }
}

/// A monotone unbounded function on natural numbers tends to infinity.
theorem monotone_unbounded_tends_to_infinity(f: Nat -> Nat) {
    is_monotone(f) and is_unbounded(f)
    implies
    tends_to_infinity(f)
} by {
    if is_monotone(f) and is_unbounded(f) {
        forall(big_n: Nat) {
            let m: Nat satisfy {
                big_n < f(m)
            }

            forall(n: Nat) {
                if m <= n {
                    f(m) <= f(n)
                    big_n < f(m)
                    big_n <= f(n)
                }
            }
        }
    }
}

/// If a sequence converges and f tends to infinity, then the composed sequence converges to the same limit.
theorem converges_compose_tends_to_infinity(a: Nat -> Real, f: Nat -> Nat) {
    converges(a) and tends_to_infinity(f)
    implies
    converges_to(compose(a, f), limit(a))
} by {
    converges_to(a, limit(a))
    let lim = limit(a)

    forall(eps: Real) {
        if eps.is_positive {
            let big_n: Nat satisfy {
                tail_bound(a, lim, big_n, eps)
            }

            let m: Nat satisfy {
                forall(n: Nat) {
                    m <= n implies big_n <= f(n)
                }
            }

            forall(n: Nat) {
                if m <= n {
                    big_n <= f(n)
                    a(f(n)).is_close(lim, eps)
                    compose(a, f)(n) = a(f(n))
                    compose(a, f)(n).is_close(lim, eps)
                }
            }

            exists(k0: Nat) {
                m <= k0 and not compose(a, f)(k0).is_close(lim, eps)
            } or tail_bound(compose(a, f), lim, m, eps)
            tail_bound(compose(a, f), lim, m, eps)
        }
    }

    converges_to(compose(a, f), lim)
}

/// If a sequence converges and f is nondecreasing and unbounded, then compose(a, f) converges to the same limit.
theorem converges_compose_nondecreasing_unbounded(a: Nat -> Real, f: Nat -> Nat) {
    converges(a) and is_nondecreasing_nat(f) and is_unbounded(f)
    implies
    converges_to(compose(a, f), limit(a))
} by {
    if converges(a) and is_nondecreasing_nat(f) and is_unbounded(f) {
        tends_to_infinity(f)
        converges_to(compose(a, f), limit(a))
    }
}

/// If a sequence converges and f is monotone and unbounded, then compose(a, f) converges to the same limit.
theorem converges_compose_monotone_unbounded(a: Nat -> Real, f: Nat -> Nat) {
    converges(a) and is_monotone(f) and is_unbounded(f)
    implies
    converges_to(compose(a, f), limit(a))
} by {
    if converges(a) and is_monotone(f) and is_unbounded(f) {
        tends_to_infinity(f)
        converges_to(compose(a, f), limit(a))
    }
}

/// True if an index map selects a subsequence.
define is_subsequence_index(f: Nat -> Nat) -> Bool {
    is_monotone(f) and is_unbounded(f)
}

/// The subsequence of `a` selected by an index map.
define subsequence(a: Nat -> Real, f: Nat -> Nat, n: Nat) -> Real {
    compose(a, f)(n)
}

/// A subsequence index is monotone.
theorem subsequence_index_is_monotone(f: Nat -> Nat) {
    is_subsequence_index(f) implies is_monotone(f)
} by {
    if is_subsequence_index(f) {
        is_monotone(f)
    }
}

/// A subsequence index is unbounded.
theorem subsequence_index_is_unbounded(f: Nat -> Nat) {
    is_subsequence_index(f) implies is_unbounded(f)
} by {
    if is_subsequence_index(f) {
        is_unbounded(f)
    }
}

/// A monotone unbounded map is a subsequence index.
theorem subsequence_index_of_monotone_unbounded(f: Nat -> Nat) {
    is_monotone(f) and is_unbounded(f) implies is_subsequence_index(f)
} by {
    if is_monotone(f) and is_unbounded(f) {
        is_subsequence_index(f)
    }
}

/// A subsequence index tends to infinity.
theorem subsequence_index_tends_to_infinity(f: Nat -> Nat) {
    is_subsequence_index(f) implies tends_to_infinity(f)
} by {
    if is_subsequence_index(f) {
        is_monotone(f)
        is_unbounded(f)
        monotone_unbounded_tends_to_infinity(f)
        tends_to_infinity(f)
    }
}

/// The selected subsequence agrees with composition by its index map.
theorem subsequence_eq_compose(a: Nat -> Real, f: Nat -> Nat) {
    subsequence(a, f) = compose(a, f)
} by {
    forall(n: Nat) {
        subsequence(a, f, n) = compose(a, f)(n)
    }
}

/// A convergent sequence has every selected subsequence converging to the same limit.
theorem converges_subsequence(a: Nat -> Real, f: Nat -> Nat) {
    converges(a) and is_subsequence_index(f)
    implies
    converges_to(subsequence(a, f), limit(a))
} by {
    if converges(a) and is_subsequence_index(f) {
        is_monotone(f)
        is_unbounded(f)
        converges_compose_monotone_unbounded(a, f)
        converges_to(compose(a, f), limit(a))
        subsequence(a, f) = compose(a, f)
        converges_to(subsequence(a, f), limit(a))
    }
}

/// A convergent sequence has every selected subsequence convergent.
theorem subsequence_converges(a: Nat -> Real, f: Nat -> Nat) {
    converges(a) and is_subsequence_index(f)
    implies
    converges(subsequence(a, f))
} by {
    if converges(a) and is_subsequence_index(f) {
        converges_to(subsequence(a, f), limit(a))
        converges(subsequence(a, f))
    }
}

/// The limit of a selected subsequence is the limit of the original sequence.
theorem limit_subsequence(a: Nat -> Real, f: Nat -> Nat) {
    converges(a) and is_subsequence_index(f)
    implies
    limit(subsequence(a, f)) = limit(a)
} by {
    if converges(a) and is_subsequence_index(f) {
        converges_to(subsequence(a, f), limit(a))
        converges(subsequence(a, f))
        converges_to(subsequence(a, f), limit(subsequence(a, f)))
        converges_to_unique(subsequence(a, f), limit(a), limit(subsequence(a, f)))
        limit(subsequence(a, f)) = limit(a)
    }
}

/// A selected subsequence has the same limit as the original sequence.
theorem subsequence_same_limit_left(a: Nat -> Real, f: Nat -> Nat) {
    converges(a) and is_subsequence_index(f)
    implies
    same_limit(subsequence(a, f), a)
} by {
    if converges(a) and is_subsequence_index(f) {
        converges(subsequence(a, f))
        limit(subsequence(a, f)) = limit(a)
        same_limit(subsequence(a, f), a)
    }
}

/// A convergent sequence has the same limit as each selected subsequence.
theorem subsequence_same_limit_right(a: Nat -> Real, f: Nat -> Nat) {
    converges(a) and is_subsequence_index(f)
    implies
    same_limit(a, subsequence(a, f))
} by {
    if converges(a) and is_subsequence_index(f) {
        converges(subsequence(a, f))
        limit(subsequence(a, f)) = limit(a)
        limit(a) = limit(subsequence(a, f))
        same_limit(a, subsequence(a, f))
    }
}

/// The function n -> k * n is nondecreasing when k >= 1.
theorem mul_nondecreasing(k: Nat) {
    Nat.1 <= k implies is_nondecreasing_nat(k.mul)
} by {
    forall(i: Nat, j: Nat) {
        if i <= j {
            k * i <= k * j
        }
    }
}

/// The function `n ↦ k * n` is monotone when `k >= 1`.
theorem mul_monotone(k: Nat) {
    Nat.1 <= k implies is_monotone(k.mul)
} by {
    if Nat.1 <= k {
        mul_nondecreasing(k)
        is_nondecreasing_nat(k.mul)
        forall(i: Nat, j: Nat) {
            if i <= j {
                k.mul(i) <= k.mul(j)
            }
        }
        monotone_from_forall(k.mul)
        is_monotone(k.mul)
    }
}

/// The function n -> k * n is unbounded when k >= 1.
theorem mul_unbounded(k: Nat) {
    Nat.1 <= k implies is_unbounded(k.mul)
} by {
    forall(bound: Nat) {
        if Nat.1 <= k {
            let n = bound.suc
            k * n >= k
            k * n >= Nat.1
            n * Nat.1 = n
            n * Nat.1 <= n * k
            n <= k * n
            bound < n
            bound.suc <= k * n
            k * n > bound
            k.mul(n) = k * n
            bound < k.mul(n)
        }
    }
}

/// The function n -> k + n is nondecreasing.
theorem add_nondecreasing(k: Nat) {
    is_nondecreasing_nat(k.add)
} by {
    forall(i: Nat, j: Nat) {
        if i <= j {
            k + i <= k + j
        }
    }
}

/// The function `n ↦ k + n` is monotone.
theorem add_monotone(k: Nat) {
    is_monotone(k.add)
} by {
    add_nondecreasing(k)
    is_nondecreasing_nat(k.add)
    forall(i: Nat, j: Nat) {
        if i <= j {
            k.add(i) <= k.add(j)
        }
    }
    monotone_from_forall(k.add)
    is_monotone(k.add)
}

/// The function n -> k + n is unbounded.
theorem add_unbounded(k: Nat) {
    is_unbounded(k.add)
} by {
    forall(bound: Nat) {
        let n: Nat satisfy {
            bound < k + n
        }
        k.add(n) = k + n
        bound < k.add(n)
    }
}

/// The index map `n ↦ k * n` selects a subsequence when `k >= 1`.
theorem mul_subsequence_index(k: Nat) {
    Nat.1 <= k implies is_subsequence_index(k.mul)
} by {
    if Nat.1 <= k {
        mul_monotone(k)
        mul_unbounded(k)
        is_subsequence_index(k.mul)
    }
}

/// The index map `n ↦ k + n` selects a subsequence.
theorem add_subsequence_index(k: Nat) {
    is_subsequence_index(k.add)
} by {
    add_monotone(k)
    add_unbounded(k)
    is_subsequence_index(k.add)
}

/// The even-index map selects a subsequence.
theorem double_subsequence_index {
    is_subsequence_index(Nat.2.mul)
} by {
    Nat.1 <= Nat.2
    mul_subsequence_index(Nat.2)
}

/// The successor index map selects a subsequence.
theorem suc_subsequence_index {
    is_subsequence_index(Nat.suc)
} by {
    Nat.1.add = Nat.suc
    add_subsequence_index(Nat.1)
}

/// If a sequence converges and k >= 1, then taking every k-th element preserves convergence.
theorem converges_compose_mul(a: Nat -> Real, k: Nat) {
    converges(a) and Nat.1 <= k
    implies
    converges_to(compose(a, k.mul), limit(a))
} by {
    Nat.1 <= k implies is_monotone(k.mul)
    Nat.1 <= k implies is_unbounded(k.mul)
    converges(a) and is_monotone(k.mul) and is_unbounded(k.mul) implies converges_to(compose(a, k.mul), limit(a))
}

/// If a sequence converges, then any tail (starting from index k) converges to the same limit.
theorem converges_compose_add(a: Nat -> Real, k: Nat) {
    converges(a)
    implies
    converges_to(compose(a, k.add), limit(a))
} by {
    is_monotone(k.add)
    is_unbounded(k.add)
    converges(a) and is_monotone(k.add) and is_unbounded(k.add) implies converges_to(compose(a, k.add), limit(a))
}

/// If a sequence converges, then taking every other element (even indices) preserves convergence.
theorem converges_compose_double(a: Nat -> Real) {
    converges(a)
    implies
    converges_to(compose(a, Nat.2.mul), limit(a))
} by {
    Nat.1 <= Nat.2
}

/// If a sequence converges, then its successor-shifted version converges to the same limit.
/// In other words, the tail starting from index 1 has the same limit as the original sequence.
theorem converges_compose_suc(a: Nat -> Real) {
    converges(a)
    implies
    converges_to(compose(a, Nat.suc), limit(a))
} by {
    // Nat.suc = Nat.1.add, so we can use converges_compose_add
    compose(a, Nat.1.add) = compose(a, Nat.suc)
    converges_to(compose(a, Nat.suc), limit(a))
}

/// If a sequence converges and `k >= 1`, then its every-`k`th subsequence converges to the same limit.
theorem converges_subsequence_mul(a: Nat -> Real, k: Nat) {
    converges(a) and Nat.1 <= k
    implies
    converges_to(subsequence(a, k.mul), limit(a))
} by {
    if converges(a) and Nat.1 <= k {
        is_subsequence_index(k.mul)
        converges_subsequence(a, k.mul)
        converges_to(subsequence(a, k.mul), limit(a))
    }
}

/// If a sequence converges, then its tail subsequence converges to the same limit.
theorem converges_subsequence_add(a: Nat -> Real, k: Nat) {
    converges(a)
    implies
    converges_to(subsequence(a, k.add), limit(a))
} by {
    if converges(a) {
        is_subsequence_index(k.add)
        converges_subsequence(a, k.add)
        converges_to(subsequence(a, k.add), limit(a))
    }
}

/// If a sequence converges, then its even-indexed subsequence converges to the same limit.
theorem converges_subsequence_double(a: Nat -> Real) {
    converges(a)
    implies
    converges_to(subsequence(a, Nat.2.mul), limit(a))
} by {
    if converges(a) {
        is_subsequence_index(Nat.2.mul)
        converges_subsequence(a, Nat.2.mul)
        converges_to(subsequence(a, Nat.2.mul), limit(a))
    }
}

/// If a sequence converges, then its successor-shifted subsequence converges to the same limit.
theorem converges_subsequence_suc(a: Nat -> Real) {
    converges(a)
    implies
    converges_to(subsequence(a, Nat.suc), limit(a))
} by {
    if converges(a) {
        is_subsequence_index(Nat.suc)
        converges_subsequence(a, Nat.suc)
        converges_to(subsequence(a, Nat.suc), limit(a))
    }
}

/// Alias endpoint: convergence is preserved by selected subsequences.
theorem subsequence_converges_to_limit(a: Nat -> Real, f: Nat -> Nat) {
    converges(a) and is_subsequence_index(f)
    implies
    converges_to(subsequence(a, f), limit(a))
} by {
    converges_subsequence(a, f)
}

/// Alias endpoint: selected subsequences converge when the original sequence converges.
theorem subsequence_of_convergent_converges(a: Nat -> Real, f: Nat -> Nat) {
    converges(a) and is_subsequence_index(f)
    implies
    converges(subsequence(a, f))
} by {
    subsequence_converges(a, f)
}

/// Alias endpoint: selected subsequences have the same limit as the original sequence.
theorem subsequence_limit_eq_original(a: Nat -> Real, f: Nat -> Nat) {
    converges(a) and is_subsequence_index(f)
    implies
    limit(subsequence(a, f)) = limit(a)
} by {
    limit_subsequence(a, f)
}

/// Alias endpoint: vanishing is preserved by absolute value.
theorem abs_of_vanishing_vanishes(a: Nat -> Real) {
    vanishes(a) implies vanishes(abs_seq(a))
} by {
    vanishes_abs_seq(a)
}

/// Alias endpoint: squeeze criterion for vanishing sequences.
theorem vanishes_by_squeeze(a: Nat -> Real, b: Nat -> Real) {
    seq_lte(const_seq(Real.0), a) and seq_lte(a, b) and vanishes(b)
    implies vanishes(a)
} by {
    vanishes_squeeze(a, b)
}

/// Alias endpoint: a sequence dominated in absolute value by a vanishing sequence also vanishes.
theorem vanishes_by_abs_domination(a: Nat -> Real, b: Nat -> Real) {
    seq_lte(abs_seq(a), abs_seq(b)) and vanishes(b)
    implies vanishes(a)
} by {
    vanishes_of_abs_lte_abs(a, b)
}

/// Alias endpoint: adding a vanishing sequence preserves the limit.
theorem same_limit_add_vanishing(a: Nat -> Real, b: Nat -> Real) {
    converges(a) and vanishes(b)
    implies same_limit(add_seq(a, b), a)
} by {
    same_limit_add_vanish_right(a, b)
}
