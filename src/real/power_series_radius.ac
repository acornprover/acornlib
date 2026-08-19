/// Radius of convergence of real power series.
///
/// The radius of convergence of Σ a(n) x^n is the supremum of the set of
/// nonnegative radii r at which the series converges absolutely:
///
///     rho = sup { r >= 0 : Σ |a(n)| r^n converges }.
///
/// The main results:
///
///   - the set of radii is nonempty: the series converges absolutely at 0
///     (ps_conv_radii_contains_zero),
///   - a radius is nonnegative, and every point of the disk lies inside the
///     radius: |x| < rho implies Σ a(n) x^n converges absolutely
///     (ps_abs_conv_inside_radius),
///   - convergence at a point x forces |x| <= rho (ps_conv_imp_radius_ub),
///   - divergence somewhere forces a finite radius to exist
///     (ps_radius_exists_of_divergence),
///   - the exponential series converges absolutely everywhere, so its set of
///     radii is unbounded above and no finite radius of convergence exists
///     (exp_radii_set_unbounded, exp_series_radius_infinite).

from nat import Nat, pow_zero, zero_or_suc, only_zero_lte_zero, alt_suc_ne_zero
from rat import Rat
from list import partial
from data.basic.set import Set
from real.real_base import Real, abs_gte_zero, lte_lt_trans, lte_trans, lte_abs, rat_between_reals, add_lte_add, lt_add_pos, gt_zero_imp_pos, pos_gt_zero
from real.real_ring import converges, limit, converges_to
from real.real_seq import eventual_eq, eventual_eq_converges
from real.abs_conv import absolutely_converges, abs_fn, absolutely_converges_imp_converges
from real.series_deep import power_series_term, power_series_abs_conv_inside_radius, sub_sub_cancel
from real.supremum import is_set_supremum, is_set_upper_bound, is_nonempty, has_upper_bound, set_member_le_supremum, set_supremum_close_from_below, completeness
from real.exp import exp_term, exp_term_abs_converges, zero_pow_pos
from real.power_series_analysis import exp_coeff, ps_term, ps_term_exp_coeff, converges_pointwise_eq
from real.integrability import lt_imp_pos_sub
from real.holder_minkowski import abs_eq_self_of_nonneg
from order import lt_iff_lte_and_ne, not_gt_imp_lte, lt_imp_lte

numerals Real
numerals Nat

// ---------------------------------------------------------------------------
// The set of radii of convergence.
// ---------------------------------------------------------------------------

/// True if the series converges absolutely at the nonnegative radius r.
define ps_conv_radii_contains(a: Nat -> Real, r: Real) -> Bool {
    Real.0 <= r and absolutely_converges(power_series_term(a, r))
}

/// The set of nonnegative radii at which the series converges absolutely.
define ps_conv_radii_set(a: Nat -> Real) -> Set[Real] {
    Set[Real].new(ps_conv_radii_contains(a))
}

/// Membership in the set of radii is exactly the defining predicate.
theorem ps_conv_radii_set_contains_iff(a: Nat -> Real, r: Real) {
    ps_conv_radii_set(a).contains(r) = ps_conv_radii_contains(a, r)
}

/// True if rho is the radius of convergence of the series, i.e. the supremum of
/// the set of nonnegative radii of absolute convergence.
define is_ps_radius(a: Nat -> Real, rho: Real) -> Bool {
    is_set_supremum(ps_conv_radii_set(a), rho)
}

// ---------------------------------------------------------------------------
// The series at zero.
// ---------------------------------------------------------------------------

/// The zero-th term of the power series at 0 is a(0).
theorem ps_term_at_zero_zero(a: Nat -> Real) {
    power_series_term(a, Real.0, Nat.0) = a(Nat.0)
} by {
    power_series_term(a, Real.0, Nat.0) = a(Nat.0) * Real.0.pow(Nat.0)
    pow_zero(Real.0)
    Real.0.pow(Nat.0) = Real.1
    a(Nat.0) * Real.1 = a(Nat.0)
    power_series_term(a, Real.0, Nat.0) = a(Nat.0)
}

/// Every later term of the power series at 0 is zero.
theorem ps_term_at_zero_tail(a: Nat -> Real, n: Nat) {
    n >= Nat.1 implies power_series_term(a, Real.0, n) = Real.0
} by {
    if n >= Nat.1 {
        power_series_term(a, Real.0, n) = a(n) * Real.0.pow(n)
        zero_pow_pos(n)
        Real.0.pow(n) = Real.0
        a(n) * Real.0 = Real.0
        power_series_term(a, Real.0, n) = Real.0
    }
}

/// The partial sums of the absolute series at 0 are eventually |a(0)|.
theorem ps_abs_partial_at_zero_suc(a: Nat -> Real, n: Nat) {
    partial(abs_fn(power_series_term(a, Real.0)), n.suc) = a(Nat.0).abs
} by {
    define p(k: Nat) -> Bool {
        partial(abs_fn(power_series_term(a, Real.0)), k.suc) = a(Nat.0).abs
    }
    // Base case k = 0: the first partial sum is |a(0)|.
    partial(abs_fn(power_series_term(a, Real.0)), Nat.1) =
        partial(abs_fn(power_series_term(a, Real.0)), Nat.0) + abs_fn(power_series_term(a, Real.0))(Nat.0)
    partial(abs_fn(power_series_term(a, Real.0)), Nat.0) = Real.0
    abs_fn(power_series_term(a, Real.0))(Nat.0) = power_series_term(a, Real.0, Nat.0).abs
    ps_term_at_zero_zero(a)
    power_series_term(a, Real.0, Nat.0) = a(Nat.0)
    power_series_term(a, Real.0, Nat.0).abs = a(Nat.0).abs
    Real.0 + a(Nat.0).abs = a(Nat.0).abs
    partial(abs_fn(power_series_term(a, Real.0)), Nat.1) = a(Nat.0).abs
    p(Nat.0)
    // Inductive step: the later terms are zero.
    forall(k: Nat) {
        if p(k) {
            p(k) = (partial(abs_fn(power_series_term(a, Real.0)), k.suc) = a(Nat.0).abs)
            partial(abs_fn(power_series_term(a, Real.0)), k.suc) = a(Nat.0).abs
            partial(abs_fn(power_series_term(a, Real.0)), k.suc.suc) =
                partial(abs_fn(power_series_term(a, Real.0)), k.suc) + abs_fn(power_series_term(a, Real.0))(k.suc)
            abs_fn(power_series_term(a, Real.0))(k.suc) = power_series_term(a, Real.0, k.suc).abs
            ps_term_at_zero_tail(a, k.suc)
            k.suc >= Nat.1
            power_series_term(a, Real.0, k.suc) = Real.0
            power_series_term(a, Real.0, k.suc).abs = Real.0.abs
            abs_eq_self_of_nonneg(Real.0)
            Real.0 >= Real.0
            Real.0.abs = Real.0
            abs_fn(power_series_term(a, Real.0))(k.suc) = Real.0
            partial(abs_fn(power_series_term(a, Real.0)), k.suc.suc) = a(Nat.0).abs + Real.0
            a(Nat.0).abs + Real.0 = a(Nat.0).abs
            partial(abs_fn(power_series_term(a, Real.0)), k.suc.suc) = a(Nat.0).abs
            p(k.suc)
        }
    }
    p(Nat.0) and forall(k: Nat) { p(k) implies p(k.suc) }
    Nat.induction(p)
    p(n)
}

/// The set of radii contains 0: the series converges absolutely at 0.
theorem ps_conv_radii_contains_zero(a: Nat -> Real) {
    ps_conv_radii_set(a).contains(Real.0)
} by {
    // The partial sums of the absolute series are eventually |a(0)|.
    define p(i: Nat) -> Bool {
        Nat.1 <= i implies partial(abs_fn(power_series_term(a, Real.0)), i) = a(Nat.0).abs
    }
    // Base case: 1 <= 0 is impossible.
    if Nat.1 <= Nat.0 {
        only_zero_lte_zero(Nat.1)
        Nat.1 = Nat.0
        alt_suc_ne_zero(Nat.0)
        Nat.1 != Nat.0
        false
    }
    p(Nat.0)
    // Inductive step: the (i+1)-st partial sum is |a(0)|.
    forall(i: Nat) {
        if p(i) {
            ps_abs_partial_at_zero_suc(a, i)
            partial(abs_fn(power_series_term(a, Real.0)), i.suc) = a(Nat.0).abs
            p(i.suc)
        }
    }
    p(Nat.0) and forall(i: Nat) { p(i) implies p(i.suc) }
    Nat.induction(p)
    forall(i: Nat) {
        p(i)
        p(i) = (Nat.1 <= i implies partial(abs_fn(power_series_term(a, Real.0)), i) = a(Nat.0).abs)
        Nat.1 <= i implies partial(abs_fn(power_series_term(a, Real.0)), i) = a(Nat.0).abs
    }
    exists(n: Nat) {
        forall(i: Nat) {
            n <= i implies partial(abs_fn(power_series_term(a, Real.0)), i) = a(Nat.0).abs
        }
    }
    eventual_eq(partial(abs_fn(power_series_term(a, Real.0))), a(Nat.0).abs)
    eventual_eq_converges(partial(abs_fn(power_series_term(a, Real.0))), a(Nat.0).abs)
    converges(partial(abs_fn(power_series_term(a, Real.0))))
    absolutely_converges(power_series_term(a, Real.0)) =
        converges(partial(abs_fn(power_series_term(a, Real.0))))
    absolutely_converges(power_series_term(a, Real.0))
    Real.0 <= Real.0
    ps_conv_radii_contains(a, Real.0) =
        (Real.0 <= Real.0 and absolutely_converges(power_series_term(a, Real.0)))
    ps_conv_radii_contains(a, Real.0)
    ps_conv_radii_set_contains_iff(a, Real.0)
    ps_conv_radii_set(a).contains(Real.0) = ps_conv_radii_contains(a, Real.0)
    ps_conv_radii_set(a).contains(Real.0)
}

/// A radius of convergence is nonnegative.
theorem ps_radius_nonneg(a: Nat -> Real, rho: Real) {
    is_ps_radius(a, rho) implies Real.0 <= rho
} by {
    if is_ps_radius(a, rho) {
        ps_conv_radii_contains_zero(a)
        set_member_le_supremum(ps_conv_radii_set(a), rho, Real.0)
        Real.0 <= rho
    }
}

// ---------------------------------------------------------------------------
// Inside the radius of convergence.
// ---------------------------------------------------------------------------

/// If |x| < rho, the series converges absolutely at x.
theorem ps_abs_conv_inside_radius(a: Nat -> Real, x: Real, rho: Real) {
    is_ps_radius(a, rho) and x.abs < rho
    implies absolutely_converges(power_series_term(a, x))
} by {
    if is_ps_radius(a, rho) and x.abs < rho {
        // eps = rho - |x| > 0, and the supremum is approached from below.
        lt_imp_pos_sub(x.abs, rho)
        Real.0 < rho - x.abs
        gt_zero_imp_pos(rho - x.abs)
        (rho - x.abs).is_positive
        set_supremum_close_from_below(ps_conv_radii_set(a), rho, rho - x.abs)
        exists(r: Real) {
            ps_conv_radii_set(a).contains(r) and rho - (rho - x.abs) < r and r <= rho
                and r.is_close(rho, rho - x.abs)
        }
        exists(r: Real) {
            ps_conv_radii_set(a).contains(r) and rho - (rho - x.abs) < r
        }
        let r: Real satisfy {
            ps_conv_radii_set(a).contains(r) and rho - (rho - x.abs) < r
        }
        // |x| = rho - (rho - |x|) < r, so r is a positive radius.
        sub_sub_cancel(rho, x.abs)
        rho - (rho - x.abs) = x.abs
        x.abs < r
        abs_gte_zero(x)
        Real.0 <= x.abs
        lte_lt_trans(Real.0, x.abs, r)
        Real.0 < r
        lt_iff_lte_and_ne[Real](Real.0, r)
        Real.0 < r = (Real.0 <= r and Real.0 != r)
        Real.0 != r
        r != Real.0
        // Membership gives absolute convergence at r.
        ps_conv_radii_set_contains_iff(a, r)
        ps_conv_radii_set(a).contains(r) = ps_conv_radii_contains(a, r)
        ps_conv_radii_contains(a, r)
        ps_conv_radii_contains(a, r) =
            (Real.0 <= r and absolutely_converges(power_series_term(a, r)))
        Real.0 <= r
        absolutely_converges(power_series_term(a, r))
        absolutely_converges_imp_converges(power_series_term(a, r))
        converges(partial(power_series_term(a, r)))
        // r = |r|, so |x| < |r|.
        abs_eq_self_of_nonneg(r)
        Real.0 <= r implies r.abs = r
        r.abs = r
        x.abs < r.abs
        // The fundamental power-series lemma.
        power_series_abs_conv_inside_radius(a, x, r)
        absolutely_converges(power_series_term(a, x))
    }
}

// ---------------------------------------------------------------------------
// The radius bounds every point of convergence.
// ---------------------------------------------------------------------------

/// Every radius strictly below |x| is a radius of absolute convergence when
/// the series converges at the nonzero point x.
theorem ps_conv_radii_contains_below(a: Nat -> Real, x: Real, r: Real) {
    converges(partial(power_series_term(a, x))) and x != Real.0
    and Real.0 <= r and r < x.abs
    implies ps_conv_radii_set(a).contains(r)
} by {
    if converges(partial(power_series_term(a, x))) and x != Real.0
        and Real.0 <= r and r < x.abs {
        abs_eq_self_of_nonneg(r)
        Real.0 <= r implies r.abs = r
        r.abs = r
        power_series_abs_conv_inside_radius(a, r, x)
        absolutely_converges(power_series_term(a, r))
        ps_conv_radii_set_contains_iff(a, r)
        ps_conv_radii_set(a).contains(r) = ps_conv_radii_contains(a, r)
        ps_conv_radii_contains(a, r) =
            (Real.0 <= r and absolutely_converges(power_series_term(a, r)))
        ps_conv_radii_contains(a, r)
        ps_conv_radii_set(a).contains(r) = ps_conv_radii_contains(a, r)
        ps_conv_radii_set(a).contains(r)
    }
}

/// If the series converges at x, then |x| does not exceed the radius.
theorem ps_conv_imp_radius_ub(a: Nat -> Real, x: Real, rho: Real) {
    converges(partial(power_series_term(a, x))) and is_ps_radius(a, rho)
    implies x.abs <= rho
} by {
    if converges(partial(power_series_term(a, x))) and is_ps_radius(a, rho) {
        ps_radius_nonneg(a, rho)
        Real.0 <= rho
        if x = Real.0 {
            x.abs = Real.0.abs
            abs_eq_self_of_nonneg(Real.0)
            Real.0 >= Real.0
            Real.0.abs = Real.0
            x.abs = Real.0
            Real.0 <= rho
            x.abs <= rho
        } else {
            // Every r with 0 <= r < |x| is a member, hence bounded by rho.
            forall(r: Real) {
                if Real.0 <= r and r < x.abs {
                    ps_conv_radii_contains_below(a, x, r)
                    ps_conv_radii_set(a).contains(r)
                    set_member_le_supremum(ps_conv_radii_set(a), rho, r)
                    r <= rho
                }
            }
            // If rho < |x|, pick r with rho < r < |x|, a contradiction.
            if rho < x.abs {
                rat_between_reals(rho, x.abs)
                let r0: Rat satisfy {
                    rho < Real.from_rat(r0) and Real.from_rat(r0) < x.abs
                }
                lte_lt_trans(Real.0, rho, Real.from_rat(r0))
                Real.0 < Real.from_rat(r0)
                lt_imp_lte(Real.0, Real.from_rat(r0))
                Real.0 <= Real.from_rat(r0)
                ps_conv_radii_contains_below(a, x, Real.from_rat(r0))
                ps_conv_radii_set(a).contains(Real.from_rat(r0))
                set_member_le_supremum(ps_conv_radii_set(a), rho, Real.from_rat(r0))
                Real.from_rat(r0) <= rho
                rho < Real.from_rat(r0)
                false
            }
            not rho < x.abs
            not_gt_imp_lte(x.abs, rho)
            x.abs <= rho
        }
    }
}

// ---------------------------------------------------------------------------
// Existence of the radius.
// ---------------------------------------------------------------------------

/// If the series does not converge absolutely at x1, a finite radius of
/// convergence exists.
theorem ps_radius_exists_of_divergence(a: Nat -> Real, x1: Real) {
    not absolutely_converges(power_series_term(a, x1))
    implies exists(rho: Real) { is_ps_radius(a, rho) }
} by {
    if not absolutely_converges(power_series_term(a, x1)) {
        // The set of radii is nonempty: it contains 0.
        ps_conv_radii_contains_zero(a)
        exists(x: Real) { ps_conv_radii_set(a).contains(x) }
        is_nonempty(ps_conv_radii_set(a))
        // Every member r satisfies r <= |x1|, since absolute convergence at r
        // would give absolute convergence at x1 when |x1| < r.
        forall(r: Real) {
            if ps_conv_radii_set(a).contains(r) {
                ps_conv_radii_set_contains_iff(a, r)
                ps_conv_radii_set(a).contains(r) = ps_conv_radii_contains(a, r)
                ps_conv_radii_contains(a, r)
                ps_conv_radii_contains(a, r) =
                    (Real.0 <= r and absolutely_converges(power_series_term(a, r)))
                Real.0 <= r
                absolutely_converges(power_series_term(a, r))
                if x1.abs < r {
                    abs_eq_self_of_nonneg(r)
                    Real.0 <= r implies r.abs = r
                    r.abs = r
                    absolutely_converges_imp_converges(power_series_term(a, r))
                    converges(partial(power_series_term(a, r)))
                    abs_gte_zero(x1)
                    Real.0 <= x1.abs
                    lte_lt_trans(Real.0, x1.abs, r)
                    Real.0 < r
                    lt_iff_lte_and_ne[Real](Real.0, r)
                    Real.0 < r = (Real.0 <= r and Real.0 != r)
                    Real.0 != r
                    r != Real.0
                    power_series_abs_conv_inside_radius(a, x1, r)
                    absolutely_converges(power_series_term(a, x1))
                    false
                }
                not x1.abs < r
                not_gt_imp_lte(r, x1.abs)
                r <= x1.abs
            }
        }
        is_set_upper_bound(ps_conv_radii_set(a), x1.abs)
        has_upper_bound(ps_conv_radii_set(a))
        completeness(ps_conv_radii_set(a))
        let rho: Real satisfy {
            is_set_supremum(ps_conv_radii_set(a), rho)
        }
        is_ps_radius(a, rho)
        exists(r1: Real) { is_ps_radius(a, r1) }
    }
}

// ---------------------------------------------------------------------------
// The exponential series has infinite radius of convergence.
// ---------------------------------------------------------------------------

/// The power-series term of the exponential coefficients is the exponential
/// term.
theorem ps_term_eq_power_series_term(a: Nat -> Real, x: Real, n: Nat) {
    ps_term(a, x, n) = power_series_term(a, x, n)
} by {
    ps_term(a, x, n) = a(n) * x.pow(n)
    power_series_term(a, x, n) = a(n) * x.pow(n)
    ps_term(a, x, n) = power_series_term(a, x, n)
}

/// The exponential series converges absolutely at every point.
theorem exp_series_abs_conv_everywhere(r: Real) {
    absolutely_converges(power_series_term(exp_coeff, r))
} by {
    forall(n: Nat) {
        ps_term(exp_coeff, r, n) = power_series_term(exp_coeff, r, n)
        ps_term_exp_coeff(r, n)
        ps_term(exp_coeff, r, n) = exp_term(r, n)
        power_series_term(exp_coeff, r, n) = exp_term(r, n)
        abs_fn(power_series_term(exp_coeff, r))(n) = power_series_term(exp_coeff, r, n).abs
        abs_fn(exp_term(r))(n) = exp_term(r, n).abs
        power_series_term(exp_coeff, r, n).abs = exp_term(r, n).abs
        abs_fn(power_series_term(exp_coeff, r))(n) = abs_fn(exp_term(r))(n)
    }
    converges_pointwise_eq(abs_fn(power_series_term(exp_coeff, r)), abs_fn(exp_term(r)))
    exp_term_abs_converges(r)
    absolutely_converges(exp_term(r)) =
        converges(partial(abs_fn(exp_term(r))))
    converges(partial(abs_fn(exp_term(r))))
    converges(partial(abs_fn(power_series_term(exp_coeff, r))))
    absolutely_converges(power_series_term(exp_coeff, r)) =
        converges(partial(abs_fn(power_series_term(exp_coeff, r))))
    absolutely_converges(power_series_term(exp_coeff, r))
}

/// The set of radii of the exponential series contains every nonnegative real.
theorem exp_radii_contains_every_nonneg(r: Real) {
    Real.0 <= r implies ps_conv_radii_set(exp_coeff).contains(r)
} by {
    if Real.0 <= r {
        exp_series_abs_conv_everywhere(r)
        ps_conv_radii_set_contains_iff(exp_coeff, r)
        ps_conv_radii_set(exp_coeff).contains(r) = ps_conv_radii_contains(exp_coeff, r)
        ps_conv_radii_contains(exp_coeff, r) =
            (Real.0 <= r and absolutely_converges(power_series_term(exp_coeff, r)))
        ps_conv_radii_contains(exp_coeff, r)
        ps_conv_radii_set(exp_coeff).contains(r) = ps_conv_radii_contains(exp_coeff, r)
        ps_conv_radii_set(exp_coeff).contains(r)
    }
}

/// The set of radii of the exponential series is unbounded above.
theorem exp_radii_set_unbounded {
    forall(m: Real) {
        exists(r: Real) {
            ps_conv_radii_set(exp_coeff).contains(r) and m < r
        }
    }
} by {
    forall(m: Real) {
        // r = |m| + 1 lies in the set and exceeds m.
        lte_abs(m)
        m <= m.abs
        Real.1.is_positive
        lt_add_pos(m.abs, Real.1)
        m.abs < m.abs + Real.1
        lte_lt_trans(m, m.abs, m.abs + Real.1)
        m < m.abs + Real.1
        abs_gte_zero(m)
        Real.0 <= m.abs
        Real.1.is_positive
        pos_gt_zero(Real.1)
        Real.1 > Real.0
        Real.0 < Real.1
        lt_imp_lte(Real.0, Real.1)
        Real.0 <= Real.1
        add_lte_add(Real.0, m.abs, Real.0, Real.1)
        Real.0 + Real.0 <= m.abs + Real.1
        Real.0 + Real.0 = Real.0
        Real.0 <= m.abs + Real.1
        exp_radii_contains_every_nonneg(m.abs + Real.1)
        ps_conv_radii_set(exp_coeff).contains(m.abs + Real.1)
        ps_conv_radii_set(exp_coeff).contains(m.abs + Real.1) and m < m.abs + Real.1
        exists(r: Real) {
            ps_conv_radii_set(exp_coeff).contains(r) and m < r
        }
    }
}

/// The exponential series has no finite radius of convergence.
theorem exp_series_radius_infinite {
    not exists(rho: Real) { is_ps_radius(exp_coeff, rho) }
} by {
    if exists(rho: Real) { is_ps_radius(exp_coeff, rho) } {
        let rho: Real satisfy { is_ps_radius(exp_coeff, rho) }
        exp_radii_set_unbounded
        exists(r: Real) {
            ps_conv_radii_set(exp_coeff).contains(r) and rho < r
        }
        let r: Real satisfy {
            ps_conv_radii_set(exp_coeff).contains(r) and rho < r
        }
        set_member_le_supremum(ps_conv_radii_set(exp_coeff), rho, r)
        r <= rho
        rho < r
        false
    }
}
