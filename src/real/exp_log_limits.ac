/// Continuity of the exponential and logarithm, and their limiting behaviour
/// along sequences.
///
/// This file defines real-valued "tends to infinity" and "tends to negative
/// infinity" predicates for sequences and proves that the exponential and the
/// logarithm preserve convergence and divergence, including the classical
/// facts `x.exp -> infinity` as `x -> infinity`, `x.exp -> 0` as
/// `x -> -infinity`, and the logarithmic analogues.

from nat import Nat, from_nat, lte_trans, nat_lte_join_left, nat_lte_join_right
from rat import Rat
from order import lt_trans, lt_of_lte_of_lt
from data.basic.functions import compose
from real.log import Real, log_value, exp_neg
from real.exp import exp_pos, exp_unbounded, exp_increasing, exp_zero, exists_nat_gt, from_nat_real_nonneg
from real.exp_log_properties import log_strictly_increasing
from real.log_exp_foundations import log_value_exp
from real.exp_inequalities import one_div_one_div
from real.real_ring import converges, limit, from_nat_is_from_rat
from real.real_seq import converges_to, tail_bound, tail_bound_implies_is_close, converges_to_imp_converges, convergent_converges_to_limit, converges_to_unique
from real.continuity_base import continuous_at, continuous_condition, continuous
from real.continuity_sequences import continuous_at_preserves_sequence_limit
from real.derivative_continuity import derivative_continuous_at
from real.derivative_basic import has_derivative_at
from real.derivative_exp_log import exp_has_derivative_at, log_value_close
from real.real_base import pos_imp_eq_abs, lte_abs, lte_lt_trans, gt_zero_imp_pos, lt_add_right, from_rat_maintains_lte, not_lt_imp_gte
from real.harmonic import real_one_div_pos, real_inverse_antitone_pos_strict, rat_from_nat_lte_of_nat_lte

numerals Real

/// True if the sequence eventually exceeds every real bound.
define seq_tends_to_infinity(a: Nat -> Real) -> Bool {
    forall(b: Real) {
        exists(n0: Nat) {
            forall(n: Nat) {
                n0 <= n implies b < a(n)
            }
        }
    }
}

/// True if the sequence eventually lies below every real bound.
define seq_tends_to_neg_infinity(a: Nat -> Real) -> Bool {
    forall(b: Real) {
        exists(n0: Nat) {
            forall(n: Nat) {
                n0 <= n implies a(n) < b
            }
        }
    }
}

/// True if the sequence is eventually positive.
define seq_eventually_positive(a: Nat -> Real) -> Bool {
    exists(n0: Nat) {
        forall(n: Nat) {
            n0 <= n implies a(n) > Real.0
        }
    }
}

// =====================================================================
// Continuity of the exponential
// =====================================================================

/// The exponential function is continuous at every point.
theorem exp_continuous_at(x: Real) {
    continuous_at(Real.exp, x)
} by {
    exp_has_derivative_at(x)
    has_derivative_at(Real.exp, x, x.exp)
    derivative_continuous_at(Real.exp, x, x.exp)
    continuous_at(Real.exp, x)
}

/// The exponential function is continuous everywhere.
theorem exp_continuous {
    continuous(Real.exp)
} by {
    forall(x: Real) {
        exp_continuous_at(x)
        continuous_at(Real.exp, x)
    }
    continuous(Real.exp) = forall(y: Real) {
        continuous_at(Real.exp, y)
    }
    continuous(Real.exp)
}

/// The exponential preserves sequence convergence.
theorem exp_preserves_converges_to(a: Nat -> Real, x: Real) {
    converges_to(a, x) implies converges_to(compose(Real.exp, a), x.exp)
} by {
    if converges_to(a, x) {
        exp_continuous_at(x)
        continuous_at(Real.exp, x)
        continuous_at_preserves_sequence_limit(Real.exp, a, x)
        converges_to(compose(Real.exp, a), x.exp)
    }
}

/// The limit of an exponential sequence is the exponential of the limit.
theorem exp_limit_of_convergent(a: Nat -> Real, x: Real) {
    converges_to(a, x) implies limit(compose(Real.exp, a)) = x.exp
} by {
    if converges_to(a, x) {
        exp_preserves_converges_to(a, x)
        converges_to(compose(Real.exp, a), x.exp)
        converges_to_imp_converges(compose(Real.exp, a), x.exp)
        converges(compose(Real.exp, a))
        convergent_converges_to_limit(compose(Real.exp, a))
        converges_to(compose(Real.exp, a), limit(compose(Real.exp, a)))
        converges_to_unique(compose(Real.exp, a), limit(compose(Real.exp, a)), x.exp)
        limit(compose(Real.exp, a)) = x.exp
    }
}

// =====================================================================
// Exponential divergence along sequences
// =====================================================================

/// The exponential of a sequence tending to infinity tends to infinity.
theorem exp_seq_tends_to_infinity(a: Nat -> Real) {
    seq_tends_to_infinity(a) implies seq_tends_to_infinity(compose(Real.exp, a))
} by {
    if seq_tends_to_infinity(a) {
        seq_tends_to_infinity(a) = forall(b: Real) {
            exists(n0: Nat) {
                forall(n: Nat) {
                    n0 <= n implies b < a(n)
                }
            }
        }
        forall(b: Real) {
            exp_unbounded(b)
            let c: Real satisfy {
                b < c.exp
            }
            exists(n0: Nat) {
                forall(n: Nat) {
                    n0 <= n implies c < a(n)
                }
            }
            let n0: Nat satisfy {
                forall(n: Nat) {
                    n0 <= n implies c < a(n)
                }
            }
            forall(n: Nat) {
                if n0 <= n {
                    c < a(n)
                    exp_increasing(c, a(n))
                    c.exp < (a(n)).exp
                    lt_trans(b, c.exp, (a(n)).exp)
                    b < (a(n)).exp
                    compose(Real.exp, a, n) = (a(n)).exp
                    b < compose(Real.exp, a, n)
                }
            }
            exists(n0b: Nat) {
                forall(n: Nat) {
                    n0b <= n implies b < compose(Real.exp, a, n)
                }
            }
        }
        seq_tends_to_infinity(compose(Real.exp, a)) = forall(b: Real) {
            exists(n0: Nat) {
                forall(n: Nat) {
                    n0 <= n implies b < compose(Real.exp, a, n)
                }
            }
        }
        seq_tends_to_infinity(compose(Real.exp, a))
    }
}

/// The exponential of a sequence tending to negative infinity tends to zero.
theorem exp_seq_tends_to_neg_infinity_zero(a: Nat -> Real) {
    seq_tends_to_neg_infinity(a) implies converges_to(compose(Real.exp, a), Real.0)
} by {
    if seq_tends_to_neg_infinity(a) {
        seq_tends_to_neg_infinity(a) = forall(b: Real) {
            exists(n0: Nat) {
                forall(n: Nat) {
                    n0 <= n implies a(n) < b
                }
            }
        }
        forall(eps: Real) {
            if eps.is_positive {
                real_one_div_pos(eps)
                Real.1 / eps > Real.0
                exp_unbounded(Real.1 / eps)
                let c: Real satisfy {
                    Real.1 / eps < c.exp
                }
                exp_neg(c)
                (-c).exp = Real.1 / c.exp
                real_inverse_antitone_pos_strict(Real.1 / eps, c.exp)
                c.exp.inverse < (Real.1 / eps).inverse
                Real.1 / c.exp = c.exp.inverse
                (Real.1 / eps).inverse = eps
                Real.1 / c.exp < eps
                (-c).exp < eps
                exists(n0: Nat) {
                    forall(n: Nat) {
                        n0 <= n implies a(n) < -c
                    }
                }
                let n0: Nat satisfy {
                    forall(n: Nat) {
                        n0 <= n implies a(n) < -c
                    }
                }
                forall(n: Nat) {
                    if n0 <= n {
                        a(n) < -c
                        exp_increasing(a(n), -c)
                        (a(n)).exp < (-c).exp
                        lt_trans((a(n)).exp, (-c).exp, eps)
                        (a(n)).exp < eps
                        compose(Real.exp, a, n) = (a(n)).exp
                        compose(Real.exp, a, n) - Real.0 = compose(Real.exp, a, n)
                        (compose(Real.exp, a, n) - Real.0).abs = compose(Real.exp, a, n).abs
                        exp_pos(a(n))
                        (a(n)).exp > Real.0
                        pos_imp_eq_abs((a(n)).exp)
                        (a(n)).exp.abs = (a(n)).exp
                        compose(Real.exp, a, n).abs = (a(n)).exp.abs
                        (compose(Real.exp, a, n) - Real.0).abs = (a(n)).exp
                        (compose(Real.exp, a, n) - Real.0).abs < eps
                        compose(Real.exp, a, n).is_close(Real.0, eps)
                    }
                }
                tail_bound(compose(Real.exp, a), Real.0, n0, eps)
            }
        }
    }
}

// =====================================================================
// Continuity of the logarithm
// =====================================================================

/// The logarithm is continuous at every positive point.
theorem log_continuous_at_pos(x: Real) {
    x > Real.0 implies continuous_at(log_value, x)
} by {
    if x > Real.0 {
        forall(eps: Real) {
            if eps.is_positive {
                log_value_close(x, eps)
                let delta: Real satisfy {
                    delta.is_positive and delta < x and forall(z: Real) {
                        z.is_close(x, delta) implies z.log.get_or_else(Real.0).is_close(x.log.get_or_else(Real.0), eps)
                    }
                }
                forall(z: Real) {
                    if z.is_close(x, delta) {
                        z.log.get_or_else(Real.0).is_close(x.log.get_or_else(Real.0), eps)
                    }
                }
                forall(x1: Real) {
                    if x1.is_close(x, delta) {
                        x1.log.get_or_else(Real.0).is_close(x.log.get_or_else(Real.0), eps)
                        log_value(x1) = x1.log.get_or_else(Real.0)
                        log_value(x) = x.log.get_or_else(Real.0)
                        log_value(x1).is_close(log_value(x), eps)
                    }
                }
                continuous_condition(log_value, x, delta, eps)
                delta.is_positive and continuous_condition(log_value, x, delta, eps)
            }
        }
    }
}

/// The logarithm preserves sequence convergence at positive limits.
theorem log_preserves_converges_to(a: Nat -> Real, x: Real) {
    converges_to(a, x) and x > Real.0 implies converges_to(compose(log_value, a), x.log.get_or_else(Real.0))
} by {
    if converges_to(a, x) and x > Real.0 {
        log_continuous_at_pos(x)
        continuous_at(log_value, x)
        continuous_at_preserves_sequence_limit(log_value, a, x)
        converges_to(compose(log_value, a), x.log.get_or_else(Real.0))
    }
}

// =====================================================================
// Logarithm divergence along sequences
// =====================================================================

/// The logarithm of a sequence tending to infinity tends to infinity.
theorem log_seq_tends_to_infinity(a: Nat -> Real) {
    seq_tends_to_infinity(a) implies seq_tends_to_infinity(compose(log_value, a))
} by {
    if seq_tends_to_infinity(a) {
        seq_tends_to_infinity(a) = forall(b: Real) {
            exists(n0: Nat) {
                forall(n: Nat) {
                    n0 <= n implies b < a(n)
                }
            }
        }
        forall(b: Real) {
            exp_pos(b)
            b.exp > Real.0
            exists(n0: Nat) {
                forall(n: Nat) {
                    n0 <= n implies b.exp < a(n)
                }
            }
            let n0: Nat satisfy {
                forall(n: Nat) {
                    n0 <= n implies b.exp < a(n)
                }
            }
            forall(n: Nat) {
                if n0 <= n {
                    b.exp < a(n)
                    lt_trans(Real.0, b.exp, a(n))
                    Real.0 < a(n)
                    a(n) > Real.0
                    log_strictly_increasing(b.exp, a(n))
                    (b.exp).log.get_or_else(Real.0) < (a(n)).log.get_or_else(Real.0)
                    log_value_exp(b)
                    (b.exp).log.get_or_else(Real.0) = b
                    b < (a(n)).log.get_or_else(Real.0)
                    compose(log_value, a, n) = log_value(a(n))
                    log_value(a(n)) = (a(n)).log.get_or_else(Real.0)
                    compose(log_value, a, n) = (a(n)).log.get_or_else(Real.0)
                    b < compose(log_value, a, n)
                }
            }
            exists(n0b: Nat) {
                forall(n: Nat) {
                    n0b <= n implies b < compose(log_value, a, n)
                }
            }
        }
        seq_tends_to_infinity(compose(log_value, a)) = forall(b: Real) {
            exists(n0: Nat) {
                forall(n: Nat) {
                    n0 <= n implies b < compose(log_value, a, n)
                }
            }
        }
        seq_tends_to_infinity(compose(log_value, a))
    }
}

/// The logarithm of a positive sequence tending to zero tends to negative
/// infinity.
theorem log_seq_tends_to_neg_infinity_zero(a: Nat -> Real) {
    converges_to(a, Real.0) and seq_eventually_positive(a)
    implies seq_tends_to_neg_infinity(compose(log_value, a))
} by {
    if converges_to(a, Real.0) and seq_eventually_positive(a) {
        converges_to(a, Real.0) = forall(eps2: Real) {
            eps2.is_positive implies exists(n: Nat) {
                tail_bound(a, Real.0, n, eps2)
            }
        }
        seq_eventually_positive(a) = exists(n2: Nat) {
            forall(n: Nat) {
                n2 <= n implies a(n) > Real.0
            }
        }
        forall(b: Real) {
            exp_pos(b)
            b.exp > Real.0
            gt_zero_imp_pos(b.exp)
            b.exp.is_positive
            exists(n1: Nat) {
                tail_bound(a, Real.0, n1, b.exp)
            }
            let n1: Nat satisfy {
                tail_bound(a, Real.0, n1, b.exp)
            }
            exists(n2: Nat) {
                forall(n: Nat) {
                    n2 <= n implies a(n) > Real.0
                }
            }
            let n2: Nat satisfy {
                forall(n: Nat) {
                    n2 <= n implies a(n) > Real.0
                }
            }
            forall(n: Nat) {
                if n1.join(n2) <= n {
                    nat_lte_join_left(n1, n2)
                    n1 <= n1.join(n2)
                    lte_trans(n1, n1.join(n2), n)
                    n1 <= n
                    tail_bound_implies_is_close(a, Real.0, n1, b.exp, n)
                    a(n).is_close(Real.0, b.exp)
                    a(n).is_close(Real.0, b.exp) = (a(n) - Real.0).abs < b.exp
                    (a(n) - Real.0).abs < b.exp
                    a(n) - Real.0 = a(n)
                    a(n).abs < b.exp
                    lte_abs(a(n))
                    a(n) <= a(n).abs
                    lte_lt_trans(a(n), a(n).abs, b.exp)
                    a(n) < b.exp
                    nat_lte_join_right(n1, n2)
                    n2 <= n1.join(n2)
                    lte_trans(n2, n1.join(n2), n)
                    n2 <= n
                    a(n) > Real.0
                    log_strictly_increasing(a(n), b.exp)
                    (a(n)).log.get_or_else(Real.0) < (b.exp).log.get_or_else(Real.0)
                    log_value_exp(b)
                    (b.exp).log.get_or_else(Real.0) = b
                    (a(n)).log.get_or_else(Real.0) < b
                    compose(log_value, a, n) = log_value(a(n))
                    log_value(a(n)) = (a(n)).log.get_or_else(Real.0)
                    compose(log_value, a, n) = (a(n)).log.get_or_else(Real.0)
                    compose(log_value, a, n) < b
                }
            }
            exists(n0: Nat) {
                forall(n: Nat) {
                    n0 <= n implies compose(log_value, a, n) < b
                }
            }
        }
        seq_tends_to_neg_infinity(compose(log_value, a)) = forall(b: Real) {
            exists(n0: Nat) {
                forall(n: Nat) {
                    n0 <= n implies compose(log_value, a, n) < b
                }
            }
        }
        seq_tends_to_neg_infinity(compose(log_value, a))
    }
}

/// The limit of a logarithm sequence is the logarithm of the limit.
theorem log_limit_of_convergent(a: Nat -> Real, x: Real) {
    converges_to(a, x) and x > Real.0 implies limit(compose(log_value, a)) = x.log.get_or_else(Real.0)
} by {
    if converges_to(a, x) and x > Real.0 {
        log_preserves_converges_to(a, x)
        converges_to(compose(log_value, a), x.log.get_or_else(Real.0))
        converges_to_imp_converges(compose(log_value, a), x.log.get_or_else(Real.0))
        converges(compose(log_value, a))
        convergent_converges_to_limit(compose(log_value, a))
        converges_to(compose(log_value, a), limit(compose(log_value, a)))
        converges_to_unique(compose(log_value, a), limit(compose(log_value, a)), x.log.get_or_else(Real.0))
        limit(compose(log_value, a)) = x.log.get_or_else(Real.0)
    }
}

/// The exponential of a sequence is eventually positive.
theorem exp_seq_eventually_positive(a: Nat -> Real) {
    seq_eventually_positive(compose(Real.exp, a))
} by {
    forall(n: Nat) {
        if Nat.0 <= n {
            exp_pos(a(n))
            (a(n)).exp > Real.0
            compose(Real.exp, a, n) = (a(n)).exp
            compose(Real.exp, a, n) > Real.0
        }
    }
    exists(n0: Nat) {
        forall(n: Nat) {
            n0 <= n implies compose(Real.exp, a, n) > Real.0
        }
    }
    seq_eventually_positive(compose(Real.exp, a)) = exists(n2: Nat) {
        forall(n: Nat) {
            n2 <= n implies compose(Real.exp, a, n) > Real.0
        }
    }
    seq_eventually_positive(compose(Real.exp, a))
}

/// The natural-number real sequence tends to infinity.
theorem from_nat_seq_tends_to_infinity {
    seq_tends_to_infinity(from_nat[Real])
} by {
    seq_tends_to_infinity(from_nat[Real]) = forall(b: Real) {
        exists(n0: Nat) {
            forall(n: Nat) {
                n0 <= n implies b < from_nat[Real](n)
            }
        }
    }
    forall(b: Real) {
        if b < Real.0 {
            forall(n: Nat) {
                if Nat.0 <= n {
                    from_nat_real_nonneg(n)
                    from_nat[Real](n) >= Real.0
                    Real.0 <= from_nat[Real](n)
                    Real.0 > b
                    b < Real.0
                    lt_of_lte_of_lt(b, Real.0, from_nat[Real](n))
                    b < from_nat[Real](n)
                }
            }
            exists(n0b: Nat) {
                forall(n: Nat) {
                    n0b <= n implies b < from_nat[Real](n)
                }
            }
        } else {
            not_lt_imp_gte(b, Real.0)
            b >= Real.0
            Real.0 < Real.1
            lt_add_right(Real.0, Real.1, b)
            Real.0 + b < Real.1 + b
            Real.0 + b = b
            b < Real.1 + b
            Real.1 + b > b
            b >= Real.0
            lt_of_lte_of_lt(Real.0, b, Real.1 + b)
            Real.0 < Real.1 + b
            Real.1 + b > Real.0
            gt_zero_imp_pos(Real.1 + b)
            (Real.1 + b).is_positive
            exists_nat_gt(Real.1 + b)
            let n0: Nat satisfy {
                Real.1 + b < from_nat[Real](n0)
            }
            forall(n: Nat) {
                if n0 <= n {
                    rat_from_nat_lte_of_nat_lte(n0, n)
                    Rat.from_nat(n0) <= Rat.from_nat(n)
                    from_rat_maintains_lte(Rat.from_nat(n0), Rat.from_nat(n))
                    Real.from_rat(Rat.from_nat(n0)) <= Real.from_rat(Rat.from_nat(n))
                    from_nat_is_from_rat(n0)
                    Real.from_rat(Rat.from_nat(n0)) = from_nat[Real](n0)
                    from_nat_is_from_rat(n)
                    Real.from_rat(Rat.from_nat(n)) = from_nat[Real](n)
                    from_nat[Real](n0) <= from_nat[Real](n)
                    Real.1 + b < from_nat[Real](n0)
                    b < Real.1 + b
                    lt_trans(b, Real.1 + b, from_nat[Real](n0))
                    b < from_nat[Real](n0)
                    lte_lt_trans(b, from_nat[Real](n0), from_nat[Real](n))
                    b < from_nat[Real](n)
                }
            }
            exists(n0b: Nat) {
                forall(n: Nat) {
                    n0b <= n implies b < from_nat[Real](n)
                }
            }
        }
    }
    seq_tends_to_infinity(from_nat[Real])
}
