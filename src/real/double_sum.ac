/// Double sums (limits of rectangular sums) for functions from (Nat, Nat) to Real.
from nat import Nat
from list import partial
from data.basic.functions import flip
from real.real_base import Real, not_lte_imp_gt
from real.rectangular_sum import rectangular_sum, row_sum, col_sum, nonneg_fn_2, rectangular_sum_nonneg, rectangular_sum_increasing_rows, rectangular_sum_increasing_cols, rectangular_sum_zero_rows, rectangular_sum_zero_cols, add_fn_2, sub_fn_2, rectangular_sum_add, rectangular_sum_sub, lte_fn_2
from real.double_limit import double_limit_condition, double_converges_to, double_converges, double_limit, doubly_increasing, double_image_is_supremum, double_is_upper_bound, double_image, doubly_increasing_bounded_converges, double_limit_eq_of_converges_to, row_limit, col_limit, row_limit_unfold, col_limit_unfold, iterated_limit_rows, iterated_limit_cols, monotone_convergence_interchange, iterated_rows_converges, iterated_cols_converges, double_limit_sub
from real.real_seq import converges, limit, converges_to, converges_to_imp_converges, converges_imp_converges_to, converges_to_unique, eventual_eq, eq_converges, eq_imp_limit, add_seq, limit_add_seq, tail_bound
from real.real_series import is_increasing, is_upper_bound, increasing_convergent_bounded_by_limit, monotone_convergence_principle, neg_seq, neg_seq_converges, neg_seq_converges_to, partial_zero, increasing_is_monotone
from real.abs_conv import sub_seq
from data.basic.set import Set
from real.supremum import is_nonempty, is_set_upper_bound, has_upper_bound, is_set_supremum, completeness
from order import is_monotone
from order import lte_trans

/// The condition that the rectangular sum of f is within eps of a for all dimensions beyond n.
define double_sum_bounded(f: (Nat, Nat) -> Real, a: Real, n: Nat, eps: Real) -> Bool {
    double_limit_condition(rectangular_sum(f), a, n, eps)
}

/// The double sum of f converges to a.
define double_sum_converges_to(f: (Nat, Nat) -> Real, a: Real) -> Bool {
    double_converges_to(rectangular_sum(f), a)
}

/// The double sum of f converges to some value.
define double_sum_converges(f: (Nat, Nat) -> Real) -> Bool {
    double_converges(rectangular_sum(f))
}

/// The limit of the double sum of f. Only meaningful when f converges.
define double_sum(f: (Nat, Nat) -> Real) -> Real {
    double_limit(rectangular_sum(f))
}

/// The infinite sum over row i: sum_{j=0}^∞ f(i, j).
/// Only meaningful when the row series converges.
define row_infinite_sum(f: (Nat, Nat) -> Real, i: Nat) -> Real {
    limit(partial(f(i)))
}

/// The infinite sum over column j: sum_{i=0}^∞ f(i, j).
/// Only meaningful when the column series converges.
define col_infinite_sum(f: (Nat, Nat) -> Real, j: Nat) -> Real {
    limit(partial(flip(f, j)))
}

/// Sequence of row infinite sums: maps each row index to its infinite sum.
define row_sums_seq(f: (Nat, Nat) -> Real, i: Nat) -> Real {
    row_infinite_sum(f, i)
}

/// Sequence of column infinite sums: maps each column index to its infinite sum.
define col_sums_seq(f: (Nat, Nat) -> Real, j: Nat) -> Real {
    col_infinite_sum(f, j)
}

/// The sum of f over an n × n square.
/// Computes sum_{i=0}^{n-1} sum_{j=0}^{n-1} f(i, j).
define square_sum(f: (Nat, Nat) -> Real, n: Nat) -> Real {
    rectangular_sum(f, n, n)
}

/// Square partial sums as a sequence in the side length.
define square_sum_seq(f: (Nat, Nat) -> Real, n: Nat) -> Real {
    square_sum(f, n)
}


/// True if all row series of f converge.
define all_rows_converge(f: (Nat, Nat) -> Real) -> Bool {
    forall(i: Nat) { converges(partial(f(i))) }
}

/// True if all column series of f converge.
define all_cols_converge(f: (Nat, Nat) -> Real) -> Bool {
    forall(j: Nat) { converges(partial(flip(f, j))) }
}

/// For nonnegative functions, partial sums of a row are increasing.
theorem row_partial_increasing(f: (Nat, Nat) -> Real, i: Nat) {
    nonneg_fn_2(f)
    implies
    is_increasing(partial(f(i)))
} by {
    if nonneg_fn_2(f) {
        forall(n: Nat) {
            Real.0 <= f(i, n)
            partial(f(i), n) <= partial(f(i), n) + f(i, n)
            partial(f(i), n.suc) = partial(f(i), n) + f(i, n)
            partial(f(i), n) <= partial(f(i), n.suc)
        }
    }
}

/// For nonnegative functions, row partial sums are monotone maps.
theorem row_partial_monotone(f: (Nat, Nat) -> Real, i: Nat) {
    nonneg_fn_2(f)
    implies
    is_monotone(partial(f(i)))
} by {
    if nonneg_fn_2(f) {
        row_partial_increasing(f, i)
        is_increasing(partial(f(i)))
        increasing_is_monotone(partial(f(i)))
        is_monotone(partial(f(i)))
    }
}

/// For nonnegative functions, partial sums of a column are increasing.
theorem col_partial_increasing(f: (Nat, Nat) -> Real, j: Nat) {
    nonneg_fn_2(f)
    implies
    is_increasing(partial(flip(f, j)))
} by {
    if nonneg_fn_2(f) {
        forall(n: Nat) {
            flip(f, j)(n) >= Real.0
            Real.0 <= flip(f, j)(n)
            partial(flip(f, j), n) <= partial(flip(f, j), n) + flip(f, j)(n)
            partial(flip(f, j), n.suc) = partial(flip(f, j), n) + flip(f, j)(n)
            partial(flip(f, j), n) <= partial(flip(f, j), n.suc)
        }
    }
}

/// For nonnegative functions, column partial sums are monotone maps.
theorem col_partial_monotone(f: (Nat, Nat) -> Real, j: Nat) {
    nonneg_fn_2(f)
    implies
    is_monotone(partial(flip(f, j)))
} by {
    if nonneg_fn_2(f) {
        col_partial_increasing(f, j)
        is_increasing(partial(flip(f, j)))
        increasing_is_monotone(partial(flip(f, j)))
        is_monotone(partial(flip(f, j)))
    }
}

/// Square partial sums of a nonnegative function form an increasing sequence.
theorem square_sum_is_increasing(f: (Nat, Nat) -> Real) {
    nonneg_fn_2(f)
    implies
    is_increasing(square_sum_seq(f))
} by {
    if nonneg_fn_2(f) {
        forall(n: Nat) {
            square_sum_seq(f, n) = square_sum(f, n)
            square_sum_seq(f, n.suc) = square_sum(f, n.suc)
            square_sum(f, n) = rectangular_sum(f, n, n)
            square_sum(f, n.suc) = rectangular_sum(f, n.suc, n.suc)
            rectangular_sum(f, n, n) <= rectangular_sum(f, n.suc, n)
            rectangular_sum(f, n.suc, n) <= rectangular_sum(f, n.suc, n.suc)
            lte_trans(rectangular_sum(f, n, n), rectangular_sum(f, n.suc, n), rectangular_sum(f, n.suc, n.suc))
            rectangular_sum(f, n, n) <= rectangular_sum(f, n.suc, n.suc)
            square_sum(f, n) <= square_sum(f, n.suc)
            square_sum_seq(f, n) <= square_sum_seq(f, n.suc)
        }
    }
}

/// Square partial sums of a nonnegative function are monotone in the side length.
theorem square_sum_is_monotone(f: (Nat, Nat) -> Real) {
    nonneg_fn_2(f)
    implies
    is_monotone(square_sum_seq(f))
} by {
    if nonneg_fn_2(f) {
        square_sum_is_increasing(f)
        is_increasing(square_sum_seq(f))
        increasing_is_monotone(square_sum_seq(f))
        is_monotone(square_sum_seq(f))
    }
}

/// Finite row sum is bounded by infinite row sum.
theorem row_sum_bounded_by_infinite(f: (Nat, Nat) -> Real, i: Nat, n: Nat) {
    nonneg_fn_2(f) and converges(partial(f(i)))
    implies
    row_sum(f, n, i) <= row_infinite_sum(f, i)
} by {
    if nonneg_fn_2(f) and converges(partial(f(i))) {
        is_upper_bound(partial(f(i)), limit(partial(f(i))))
        row_sum(f, n, i) = partial(f(i), n)
        row_infinite_sum(f, i) = limit(partial(f(i)))
        partial(f(i), n) <= limit(partial(f(i)))
        row_sum(f, n, i) <= row_infinite_sum(f, i)
    }
}

/// Finite column sum is bounded by infinite column sum.
theorem col_sum_bounded_by_infinite(f: (Nat, Nat) -> Real, j: Nat, m: Nat) {
    nonneg_fn_2(f) and converges(partial(flip(f, j)))
    implies
    col_sum(f, m, j) <= col_infinite_sum(f, j)
} by {
    if nonneg_fn_2(f) and converges(partial(flip(f, j))) {
        is_upper_bound(partial(flip(f, j)), limit(partial(flip(f, j))))
        col_sum(f, m, j) = partial(flip(f, j), m)
        col_infinite_sum(f, j) = limit(partial(flip(f, j)))
        partial(flip(f, j), m) <= limit(partial(flip(f, j)))
        col_sum(f, m, j) <= col_infinite_sum(f, j)
    }
}

/// Rectangular partial sum is bounded by sum of row infinite sums.
/// If a_{m,n} ≥ 0 and b_i = sum_{j=0}^∞ a_{i,j}, then
/// sum_{i=0}^{m-1} sum_{j=0}^{n-1} a_{i,j} ≤ sum_{i=0}^{m-1} b_i.
theorem rectangular_sum_bounded_by_row_sums(f: (Nat, Nat) -> Real, m: Nat, n: Nat) {
    nonneg_fn_2(f) and all_rows_converge(f)
    implies
    rectangular_sum(f, m, n) <= partial(row_sums_seq(f), m)
} by {
    define p(k: Nat) -> Bool {
        nonneg_fn_2(f) and all_rows_converge(f)
        implies
        rectangular_sum(f, k, n) <= partial(row_sums_seq(f), k)
    }

    if nonneg_fn_2(f) and all_rows_converge(f) {
        rectangular_sum(f, Nat.0, n) = partial(row_sums_seq(f), Nat.0)
        rectangular_sum(f, Nat.0, n) <= partial(row_sums_seq(f), Nat.0)
    }

    p(Nat.0)

    forall(k: Nat) {
        if p(k) {
            if nonneg_fn_2(f) and all_rows_converge(f) {
                rectangular_sum(f, k.suc, n) = rectangular_sum(f, k, n) + row_sum(f, n, k)
                rectangular_sum(f, k, n) <= partial(row_sums_seq(f), k)
                converges(partial(f(k)))
                row_sum(f, n, k) <= row_infinite_sum(f, k)
                row_sums_seq(f, k) = row_infinite_sum(f, k)
                row_sum(f, n, k) <= row_sums_seq(f, k)
                rectangular_sum(f, k, n) + row_sum(f, n, k) <= partial(row_sums_seq(f), k) + row_sums_seq(f, k)
                partial(row_sums_seq(f), k.suc) = partial(row_sums_seq(f), k) + row_sums_seq(f, k)
                rectangular_sum(f, k.suc, n) <= partial(row_sums_seq(f), k.suc)
            }
            p(k.suc)
        }
    }

    p(m)
}

/// Rectangular partial sum is bounded by sum of column infinite sums.
/// If a_{m,n} ≥ 0 and c_j = sum_{i=0}^∞ a_{i,j}, then
/// sum_{i=0}^{m-1} sum_{j=0}^{n-1} a_{i,j} ≤ sum_{j=0}^{n-1} c_j.
theorem rectangular_sum_bounded_by_col_sums(f: (Nat, Nat) -> Real, m: Nat, n: Nat) {
    nonneg_fn_2(f) and all_cols_converge(f)
    implies
    rectangular_sum(f, m, n) <= partial(col_sums_seq(f), n)
} by {
    define p(k: Nat) -> Bool {
        nonneg_fn_2(f) and all_cols_converge(f)
        implies
        rectangular_sum(f, m, k) <= partial(col_sums_seq(f), k)
    }

    if nonneg_fn_2(f) and all_cols_converge(f) {
        rectangular_sum(f, m, Nat.0) = partial(col_sums_seq(f), Nat.0)
        rectangular_sum(f, m, Nat.0) <= partial(col_sums_seq(f), Nat.0)
    }

    p(Nat.0)

    forall(k: Nat) {
        if p(k) {
            if nonneg_fn_2(f) and all_cols_converge(f) {
                rectangular_sum(f, m, k.suc) = rectangular_sum(f, m, k) + col_sum(f, m, k)
                rectangular_sum(f, m, k) <= partial(col_sums_seq(f), k)
                converges(partial(flip(f, k)))
                col_sum(f, m, k) <= col_infinite_sum(f, k)
                col_sums_seq(f, k) = col_infinite_sum(f, k)
                col_sum(f, m, k) <= col_sums_seq(f, k)
                rectangular_sum(f, m, k) + col_sum(f, m, k) <= partial(col_sums_seq(f), k) + col_sums_seq(f, k)
                partial(col_sums_seq(f), k.suc) = partial(col_sums_seq(f), k) + col_sums_seq(f, k)
                rectangular_sum(f, m, k.suc) <= partial(col_sums_seq(f), k.suc)
            }
            p(k.suc)
        }
    }

    p(n)
}

/// Extending the rectangle by one row adds the sum of that row.
theorem rectangular_sum_succ_row(f: (Nat, Nat) -> Real, k: Nat, n: Nat) {
    rectangular_sum(f, k.suc, n) = rectangular_sum(f, k, n) + row_sum(f, n, k)
} by {
    rectangular_sum(f, k.suc, n) = partial(row_sum(f, n), k.suc)
    partial(row_sum(f, n), k.suc) = partial(row_sum(f, n), k) + row_sum(f, n, k)
    rectangular_sum(f, k, n) = partial(row_sum(f, n), k)
}

/// Extending the rectangle by one column adds the sum of that column.
theorem rectangular_sum_succ_col(f: (Nat, Nat) -> Real, m: Nat, k: Nat) {
    rectangular_sum(f, m, k.suc) = rectangular_sum(f, m, k) + col_sum(f, m, k)
} by {
    rectangular_sum(f, m, k.suc) = partial(col_sum(f, m), k.suc)
    partial(col_sum(f, m), k.suc) = partial(col_sum(f, m), k) + col_sum(f, m, k)
    rectangular_sum(f, m, k) = partial(col_sum(f, m), k)
}

/// A finite row sum is bounded above by the supremum of rectangular sums.
theorem row_sum_bounded_by_supremum(f: (Nat, Nat) -> Real, s: Real, i: Nat, n: Nat) {
    nonneg_fn_2(f) and double_image_is_supremum(rectangular_sum(f), s)
    implies
    row_sum(f, n, i) <= s
} by {
    if nonneg_fn_2(f) and double_image_is_supremum(rectangular_sum(f), s) {
        rectangular_sum(f, i, n) >= Real.0
        Real.0 + row_sum(f, n, i) <= rectangular_sum(f, i, n) + row_sum(f, n, i)
        row_sum(f, n, i) <= rectangular_sum(f, i, n) + row_sum(f, n, i)
        rectangular_sum(f, i.suc, n) = rectangular_sum(f, i, n) + row_sum(f, n, i)
        row_sum(f, n, i) <= rectangular_sum(f, i.suc, n)
        double_image_is_supremum(rectangular_sum(f), s)
        double_is_upper_bound(rectangular_sum(f), s)
        rectangular_sum(f, i.suc, n) <= s
        row_sum(f, n, i) <= s
    }
}

/// A finite column sum is bounded above by the supremum of rectangular sums.
theorem col_sum_bounded_by_supremum(f: (Nat, Nat) -> Real, s: Real, j: Nat, m: Nat) {
    nonneg_fn_2(f) and double_image_is_supremum(rectangular_sum(f), s)
    implies
    col_sum(f, m, j) <= s
} by {
    if nonneg_fn_2(f) and double_image_is_supremum(rectangular_sum(f), s) {
        rectangular_sum(f, m, j) >= Real.0
        Real.0 + col_sum(f, m, j) <= rectangular_sum(f, m, j) + col_sum(f, m, j)
        col_sum(f, m, j) <= rectangular_sum(f, m, j) + col_sum(f, m, j)
        rectangular_sum(f, m, j.suc) = rectangular_sum(f, m, j) + col_sum(f, m, j)
        col_sum(f, m, j) <= rectangular_sum(f, m, j.suc)
        double_image_is_supremum(rectangular_sum(f), s)
        double_is_upper_bound(rectangular_sum(f), s)
        rectangular_sum(f, m, j.suc) <= s
        col_sum(f, m, j) <= s
    }
}

/// Each row sequence is bounded above by the supremum of rectangular sums.
theorem row_partial_bounded_by_supremum(f: (Nat, Nat) -> Real, s: Real, i: Nat) {
    nonneg_fn_2(f) and double_image_is_supremum(rectangular_sum(f), s)
    implies
    is_upper_bound(partial(f(i)), s)
} by {
    if nonneg_fn_2(f) and double_image_is_supremum(rectangular_sum(f), s) {
        forall(n: Nat) {
            row_sum(f, n, i) <= s
            partial(f(i), n) = row_sum(f, n, i)
            partial(f(i), n) <= s
        }
        is_upper_bound(partial(f(i)), s)
    }
}

/// Each column sequence is bounded above by the supremum of rectangular sums.
theorem col_partial_bounded_by_supremum(f: (Nat, Nat) -> Real, s: Real, j: Nat) {
    nonneg_fn_2(f) and double_image_is_supremum(rectangular_sum(f), s)
    implies
    is_upper_bound(partial(flip(f, j)), s)
} by {
    if nonneg_fn_2(f) and double_image_is_supremum(rectangular_sum(f), s) {
        forall(m: Nat) {
            col_sum(f, m, j) <= s
            partial(flip(f, j), m) = col_sum(f, m, j)
            partial(flip(f, j), m) <= s
        }
        is_upper_bound(partial(flip(f, j)), s)
    }
}

/// Each row series converges under the Tonelli hypotheses.
theorem row_partial_converges_of_supremum(f: (Nat, Nat) -> Real, s: Real, i: Nat) {
    nonneg_fn_2(f) and double_image_is_supremum(rectangular_sum(f), s)
    implies
    converges(partial(f(i)))
} by {
    if nonneg_fn_2(f) and double_image_is_supremum(rectangular_sum(f), s) {
        is_increasing(partial(f(i)))
        is_upper_bound(partial(f(i)), s)
        converges(partial(f(i)))
    }
}

/// Each column series converges under the Tonelli hypotheses.
theorem col_partial_converges_of_supremum(f: (Nat, Nat) -> Real, s: Real, j: Nat) {
    nonneg_fn_2(f) and double_image_is_supremum(rectangular_sum(f), s)
    implies
    converges(partial(flip(f, j)))
} by {
    if nonneg_fn_2(f) and double_image_is_supremum(rectangular_sum(f), s) {
        is_increasing(partial(flip(f, j)))
        is_upper_bound(partial(flip(f, j)), s)
        converges(partial(flip(f, j)))
    }
}

/// All row series converge under the Tonelli hypotheses.
theorem all_rows_converge_of_supremum(f: (Nat, Nat) -> Real, s: Real) {
    nonneg_fn_2(f) and double_image_is_supremum(rectangular_sum(f), s)
    implies
    all_rows_converge(f)
} by {
    if nonneg_fn_2(f) and double_image_is_supremum(rectangular_sum(f), s) {
        forall(i: Nat) {
            converges(partial(f(i)))
        }
        all_rows_converge(f)
    }
}

/// All column series converge under the Tonelli hypotheses.
theorem all_cols_converge_of_supremum(f: (Nat, Nat) -> Real, s: Real) {
    nonneg_fn_2(f) and double_image_is_supremum(rectangular_sum(f), s)
    implies
    all_cols_converge(f)
} by {
    if nonneg_fn_2(f) and double_image_is_supremum(rectangular_sum(f), s) {
        forall(j: Nat) {
            converges(partial(flip(f, j)))
        }
        all_cols_converge(f)
    }
}

/// The row-wise limit of rectangular sums equals the partial sum of row infinite sums.
theorem row_limit_rectangular_sum(f: (Nat, Nat) -> Real, m: Nat) {
    all_rows_converge(f)
    implies
    row_limit(rectangular_sum(f), m) = partial(row_sums_seq(f), m)
} by {
    if all_rows_converge(f) {
        define p(k: Nat) -> Bool {
            converges_to(rectangular_sum(f)(k), partial(row_sums_seq(f), k))
        }

        forall(n: Nat) {
            rectangular_sum(f)(Nat.0, n) = rectangular_sum(f, Nat.0, n)
            rectangular_sum(f, Nat.0, n) = Real.0
        }
        let n0 = Nat.0
        forall(n2: Nat) {
            if n0 <= n2 {
                rectangular_sum(f)(Nat.0, n2) = rectangular_sum(f, Nat.0, n2)
                rectangular_sum(f, Nat.0, n2) = Real.0
                rectangular_sum(f)(Nat.0, n2) = Real.0
            }
        }
        exists(n_witness: Nat) {
            forall(n2: Nat) {
                n_witness <= n2 implies rectangular_sum(f)(Nat.0, n2) = Real.0
            }
        }
        eventual_eq(rectangular_sum(f)(Nat.0), Real.0)
        converges(rectangular_sum(f)(Nat.0))
        limit(rectangular_sum(f)(Nat.0)) = Real.0
        converges_to(rectangular_sum(f)(Nat.0), limit(rectangular_sum(f)(Nat.0)))
        converges_to(rectangular_sum(f)(Nat.0), Real.0)
        p(Nat.0)

        forall(k: Nat) {
            if p(k) {
                converges(rectangular_sum(f)(k))
                converges_to(rectangular_sum(f)(k), limit(rectangular_sum(f)(k)))
                limit(rectangular_sum(f)(k)) = partial(row_sums_seq(f), k)

                all_rows_converge(f)
                converges(partial(f(k)))
                converges_to(partial(f(k)), limit(partial(f(k))))
                row_infinite_sum(f, k) = limit(partial(f(k)))
                converges_to(partial(f(k)), row_infinite_sum(f, k))

                forall(n: Nat) {
                    rectangular_sum(f, k.suc, n) = rectangular_sum(f, k, n) + row_sum(f, n, k)
                    rectangular_sum(f)(k.suc, n) = rectangular_sum(f, k.suc, n)
                    rectangular_sum(f)(k, n) = rectangular_sum(f, k, n)
                    row_sum(f, n, k) = partial(f(k), n)
                    rectangular_sum(f)(k.suc, n) = rectangular_sum(f)(k, n) + partial(f(k), n)
                    add_seq(rectangular_sum(f)(k), partial(f(k)), n) = rectangular_sum(f)(k, n) + partial(f(k), n)
                    rectangular_sum(f)(k.suc, n) = add_seq(rectangular_sum(f)(k), partial(f(k)), n)
                }
                rectangular_sum(f)(k.suc) = add_seq(rectangular_sum(f)(k), partial(f(k)))

                converges_to(add_seq(rectangular_sum(f)(k), partial(f(k))), limit(rectangular_sum(f)(k)) + limit(partial(f(k))))
                converges_to(add_seq(rectangular_sum(f)(k), partial(f(k))), partial(row_sums_seq(f), k) + row_infinite_sum(f, k))

                forall(n: Nat) {
                    rectangular_sum(f)(k.suc, n) = add_seq(rectangular_sum(f)(k), partial(f(k)), n)
                    add_seq(rectangular_sum(f)(k), partial(f(k)), n) = rectangular_sum(f)(k.suc, n)
                }
                add_seq(rectangular_sum(f)(k), partial(f(k))) = rectangular_sum(f)(k.suc)
                converges_to(rectangular_sum(f)(k.suc), partial(row_sums_seq(f), k) + row_infinite_sum(f, k))

                row_sums_seq(f, k) = row_infinite_sum(f, k)
                partial(row_sums_seq(f), k.suc) = partial(row_sums_seq(f), k) + row_sums_seq(f, k)
                converges_to(rectangular_sum(f)(k.suc), partial(row_sums_seq(f), k.suc))

                p(k.suc)
            }
        }

        p(m)
        converges(rectangular_sum(f)(m))
        row_limit(rectangular_sum(f), m) = limit(rectangular_sum(f)(m))
        converges_to(rectangular_sum(f)(m), partial(row_sums_seq(f), m))
        converges_to(rectangular_sum(f)(m), limit(rectangular_sum(f)(m)))
        row_limit(rectangular_sum(f), m) = partial(row_sums_seq(f), m)
    }
}
/// The column-wise limit of rectangular sums equals the partial sum of column infinite sums.
theorem col_limit_rectangular_sum(f: (Nat, Nat) -> Real, n: Nat) {
    all_cols_converge(f)
    implies
    col_limit(rectangular_sum(f), n) = partial(col_sums_seq(f), n)
} by {
    if all_cols_converge(f) {
        define p(k: Nat) -> Bool {
            converges_to(flip(rectangular_sum(f))(k), partial(col_sums_seq(f), k))
        }

        forall(m2: Nat) {
            flip(rectangular_sum(f))(Nat.0, m2) = rectangular_sum(f, m2, Nat.0)
            rectangular_sum(f, m2, Nat.0) = Real.0
        }
        let m0 = Nat.0
        forall(m2: Nat) {
            if m0 <= m2 {
                flip(rectangular_sum(f))(Nat.0, m2) = rectangular_sum(f, m2, Nat.0)
                rectangular_sum(f, m2, Nat.0) = Real.0
                flip(rectangular_sum(f))(Nat.0, m2) = Real.0
            }
        }
        exists(m_witness: Nat) {
            forall(m2: Nat) {
                m_witness <= m2 implies flip(rectangular_sum(f))(Nat.0, m2) = Real.0
            }
        }
        eventual_eq(flip(rectangular_sum(f))(Nat.0), Real.0)
        converges(flip(rectangular_sum(f))(Nat.0))
        limit(flip(rectangular_sum(f))(Nat.0)) = Real.0
        converges_to(flip(rectangular_sum(f))(Nat.0), limit(flip(rectangular_sum(f))(Nat.0)))
        converges_to(flip(rectangular_sum(f))(Nat.0), Real.0)
        p(Nat.0)

        forall(k: Nat) {
            if p(k) {
                converges(flip(rectangular_sum(f))(k))
                converges_to(flip(rectangular_sum(f))(k), limit(flip(rectangular_sum(f))(k)))
                limit(flip(rectangular_sum(f))(k)) = partial(col_sums_seq(f), k)

                all_cols_converge(f)
                converges(partial(flip(f, k)))
                converges_to(partial(flip(f, k)), limit(partial(flip(f, k))))
                col_infinite_sum(f, k) = limit(partial(flip(f, k)))
                converges_to(partial(flip(f, k)), col_infinite_sum(f, k))

                forall(m2: Nat) {
                    rectangular_sum(f, m2, k.suc) = rectangular_sum(f, m2, k) + col_sum(f, m2, k)
                    flip(rectangular_sum(f))(k.suc, m2) = rectangular_sum(f, m2, k.suc)
                    flip(rectangular_sum(f))(k, m2) = rectangular_sum(f, m2, k)
                    col_sum(f, m2, k) = partial(flip(f, k), m2)
                    flip(rectangular_sum(f))(k.suc, m2) = flip(rectangular_sum(f))(k, m2) + partial(flip(f, k), m2)
                    add_seq(flip(rectangular_sum(f))(k), partial(flip(f, k)), m2) = flip(rectangular_sum(f))(k, m2) + partial(flip(f, k), m2)
                    flip(rectangular_sum(f))(k.suc, m2) = add_seq(flip(rectangular_sum(f))(k), partial(flip(f, k)), m2)
                }
                flip(rectangular_sum(f))(k.suc) = add_seq(flip(rectangular_sum(f))(k), partial(flip(f, k)))

                converges_to(add_seq(flip(rectangular_sum(f))(k), partial(flip(f, k))), limit(flip(rectangular_sum(f))(k)) + limit(partial(flip(f, k))))
                converges_to(add_seq(flip(rectangular_sum(f))(k), partial(flip(f, k))), partial(col_sums_seq(f), k) + col_infinite_sum(f, k))

                forall(m2: Nat) {
                    flip(rectangular_sum(f))(k.suc, m2) = add_seq(flip(rectangular_sum(f))(k), partial(flip(f, k)), m2)
                    add_seq(flip(rectangular_sum(f))(k), partial(flip(f, k)), m2) = flip(rectangular_sum(f))(k.suc, m2)
                }
                add_seq(flip(rectangular_sum(f))(k), partial(flip(f, k))) = flip(rectangular_sum(f))(k.suc)
                converges_to(flip(rectangular_sum(f))(k.suc), partial(col_sums_seq(f), k) + col_infinite_sum(f, k))

                col_sums_seq(f, k) = col_infinite_sum(f, k)
                partial(col_sums_seq(f), k.suc) = partial(col_sums_seq(f), k) + col_sums_seq(f, k)
                converges_to(flip(rectangular_sum(f))(k.suc), partial(col_sums_seq(f), k.suc))

                p(k.suc)
            }
        }

        p(n)
        converges(flip(rectangular_sum(f))(n))
        col_limit(rectangular_sum(f), n) = limit(flip(rectangular_sum(f))(n))
        converges_to(flip(rectangular_sum(f))(n), partial(col_sums_seq(f), n))
        converges_to(flip(rectangular_sum(f))(n), limit(flip(rectangular_sum(f))(n)))
        col_limit(rectangular_sum(f), n) = partial(col_sums_seq(f), n)
    }
}

/// The iterated row sum equals the iterated limit over rows of rectangular sums.
theorem iterated_row_sum_eq_limit(f: (Nat, Nat) -> Real) {
    all_rows_converge(f)
    implies
    iterated_limit_rows(rectangular_sum(f)) = limit(partial(row_sums_seq(f)))
} by {
    if all_rows_converge(f) {
        forall(m: Nat) {
            row_limit(rectangular_sum(f), m) = partial(row_sums_seq(f), m)
        }
        iterated_limit_rows(rectangular_sum(f)) = limit(row_limit(rectangular_sum(f)))
        row_limit(rectangular_sum(f)) = partial(row_sums_seq(f))
        iterated_limit_rows(rectangular_sum(f)) = limit(partial(row_sums_seq(f)))
    }
}

/// The iterated column sum equals the iterated limit over columns of rectangular sums.
theorem iterated_col_sum_eq_limit(f: (Nat, Nat) -> Real) {
    all_cols_converge(f)
    implies
    iterated_limit_cols(rectangular_sum(f)) = limit(partial(col_sums_seq(f)))
} by {
    if all_cols_converge(f) {
        forall(n: Nat) {
            col_limit(rectangular_sum(f), n) = partial(col_sums_seq(f), n)
        }
        iterated_limit_cols(rectangular_sum(f)) = limit(col_limit(rectangular_sum(f)))
        col_limit(rectangular_sum(f)) = partial(col_sums_seq(f))
        iterated_limit_cols(rectangular_sum(f)) = limit(partial(col_sums_seq(f)))
    }
}
/// For nonnegative functions, rectangular sums are doubly increasing.
/// That is, rectangular_sum(f, m, n) increases as m or n increases.
theorem rectangular_sum_doubly_increasing(f: (Nat, Nat) -> Real) {
    nonneg_fn_2(f)
    implies
    doubly_increasing(rectangular_sum(f))
} by {
    if nonneg_fn_2(f) {
        forall(m: Nat, n: Nat) {
            rectangular_sum(f, m, n) <= rectangular_sum(f, m.suc, n)

            rectangular_sum(f, m, n) <= rectangular_sum(f, m, n.suc)
            rectangular_sum(f, m, n) <= rectangular_sum(f, m, n.suc) and rectangular_sum(f, m, n) <= rectangular_sum(f, m.suc, n)
        }
        doubly_increasing(rectangular_sum(f))
    }
}

/// Limit of rectangular partial sums equals their supremum.
/// For a nonnegative function f, if s is the supremum of all rectangular
/// partial sums, then the double sum (limit as M,N→∞) equals s.
theorem double_sum_limit_eq_supremum(f: (Nat, Nat) -> Real, s: Real) {
    nonneg_fn_2(f) and double_image_is_supremum(rectangular_sum(f), s)
    implies
    double_sum(f) = s
} by {
    if nonneg_fn_2(f) and double_image_is_supremum(rectangular_sum(f), s) {
        doubly_increasing(rectangular_sum(f))

        double_converges_to(rectangular_sum(f), s)

        double_limit(rectangular_sum(f)) = s

        double_sum(f) = double_limit(rectangular_sum(f))
        double_sum(f) = s
    }
}

/// Tonelli's theorem with explicit row and column convergence hypotheses.
/// The double sum, row-iterated sum, and column-iterated sum all coincide.
theorem tonelli_double_sum_aux(f: (Nat, Nat) -> Real, s: Real) {
    nonneg_fn_2(f) and all_rows_converge(f) and all_cols_converge(f) and double_image_is_supremum(rectangular_sum(f), s)
    implies
    double_sum(f) = limit(partial(row_sums_seq(f))) and
    double_sum(f) = limit(partial(col_sums_seq(f))) and
    limit(partial(row_sums_seq(f))) = limit(partial(col_sums_seq(f)))
} by {
    if nonneg_fn_2(f) and all_rows_converge(f) and all_cols_converge(f) and double_image_is_supremum(rectangular_sum(f), s) {
        // Rectangular sums are doubly increasing.
        doubly_increasing(rectangular_sum(f))

        // Apply monotone convergence to equate all limit orders.
        double_limit(rectangular_sum(f)) = s
        iterated_limit_rows(rectangular_sum(f)) = s
        iterated_limit_cols(rectangular_sum(f)) = s
        double_limit(rectangular_sum(f)) = iterated_limit_rows(rectangular_sum(f))
        double_limit(rectangular_sum(f)) = iterated_limit_cols(rectangular_sum(f))
        iterated_limit_rows(rectangular_sum(f)) = iterated_limit_cols(rectangular_sum(f))

        // Express the iterated limits as iterated sums.
        all_rows_converge(f)
        iterated_limit_rows(rectangular_sum(f)) = limit(partial(row_sums_seq(f)))
        limit(partial(row_sums_seq(f))) = iterated_limit_rows(rectangular_sum(f))
        iterated_limit_rows(rectangular_sum(f)) = s
        limit(partial(row_sums_seq(f))) = s

        all_cols_converge(f)
        iterated_limit_cols(rectangular_sum(f)) = limit(partial(col_sums_seq(f)))
        limit(partial(col_sums_seq(f))) = iterated_limit_cols(rectangular_sum(f))
        iterated_limit_cols(rectangular_sum(f)) = s
        limit(partial(col_sums_seq(f))) = s

        // The double sum is the double limit of rectangular sums.
        double_sum(f) = double_limit(rectangular_sum(f))
        double_limit(rectangular_sum(f)) = s
        double_sum(f) = s

        double_limit(rectangular_sum(f)) = iterated_limit_rows(rectangular_sum(f))
        double_sum(f) = iterated_limit_rows(rectangular_sum(f))
        iterated_limit_rows(rectangular_sum(f)) = limit(partial(row_sums_seq(f)))
        double_sum(f) = limit(partial(row_sums_seq(f)))

        double_limit(rectangular_sum(f)) = iterated_limit_cols(rectangular_sum(f))
        double_sum(f) = iterated_limit_cols(rectangular_sum(f))
        iterated_limit_cols(rectangular_sum(f)) = limit(partial(col_sums_seq(f)))
        double_sum(f) = limit(partial(col_sums_seq(f)))

        // Conclude the equality of all three sums.
        limit(partial(row_sums_seq(f))) = limit(partial(col_sums_seq(f)))
    }
}

/// Tonelli's theorem: nonnegative terms allow exchanging all orders of summation.
/// The double sum, row-iterated sum, and column-iterated sum all coincide.
theorem tonelli_double_sum(f: (Nat, Nat) -> Real, s: Real) {
    nonneg_fn_2(f) and double_image_is_supremum(rectangular_sum(f), s)
    implies
    double_sum(f) = limit(partial(row_sums_seq(f))) and
    double_sum(f) = limit(partial(col_sums_seq(f))) and
    limit(partial(row_sums_seq(f))) = limit(partial(col_sums_seq(f)))
} by {
    if nonneg_fn_2(f) and double_image_is_supremum(rectangular_sum(f), s) {
        all_rows_converge(f)
        all_cols_converge(f)

        double_sum(f) = limit(partial(row_sums_seq(f)))
        double_sum(f) = limit(partial(col_sums_seq(f)))
        limit(partial(row_sums_seq(f))) = limit(partial(col_sums_seq(f)))
    }
}

/// The absolute value of a two-variable function applied pointwise.
define abs_fn_2(f: (Nat, Nat) -> Real, m: Nat, n: Nat) -> Real {
    f(m, n).abs
}

/// The positive part of a two-variable function: max(f, 0).
define pos_part_2(f: (Nat, Nat) -> Real, m: Nat, n: Nat) -> Real {
    f(m, n).max(Real.0)
}

/// The negative part of a two-variable function: max(-f, 0).
define neg_part_2(f: (Nat, Nat) -> Real, m: Nat, n: Nat) -> Real {
    (-f(m, n)).max(Real.0)
}

/// The positive part is nonnegative.
theorem pos_part_2_nonneg(f: (Nat, Nat) -> Real) {
    nonneg_fn_2(pos_part_2(f))
} by {
    forall(m: Nat, n: Nat) {
        pos_part_2(f, m, n) = f(m, n).max(Real.0)
        f(m, n).max(Real.0) >= Real.0
        pos_part_2(f, m, n) >= Real.0
    }
}

/// The negative part is nonnegative.
theorem neg_part_2_nonneg(f: (Nat, Nat) -> Real) {
    nonneg_fn_2(neg_part_2(f))
} by {
    forall(m: Nat, n: Nat) {
        neg_part_2(f, m, n) = (-f(m, n)).max(Real.0)
        (-f(m, n)).max(Real.0) >= Real.0
        neg_part_2(f, m, n) >= Real.0
    }
}

/// A function equals its positive part minus its negative part.
theorem pos_neg_decomposition(f: (Nat, Nat) -> Real, m: Nat, n: Nat) {
    f(m, n) = pos_part_2(f, m, n) - neg_part_2(f, m, n)
} by {
    pos_part_2(f, m, n) = f(m, n).max(Real.0)
    neg_part_2(f, m, n) = (-f(m, n)).max(Real.0)
    if f(m, n).is_negative {
        f(m, n).max(Real.0) = Real.0
        pos_part_2(f, m, n) = Real.0
        (-f(m, n)).max(Real.0) = -f(m, n)
        neg_part_2(f, m, n) = -f(m, n)
        pos_part_2(f, m, n) - neg_part_2(f, m, n) = Real.0 - (-f(m, n))
        Real.0 - (-f(m, n)) = f(m, n)
        f(m, n) = pos_part_2(f, m, n) - neg_part_2(f, m, n)
    } else {
        f(m, n).max(Real.0) = f(m, n)
        pos_part_2(f, m, n) = f(m, n)
        (-f(m, n)).max(Real.0) = Real.0
        neg_part_2(f, m, n) = Real.0
        pos_part_2(f, m, n) - neg_part_2(f, m, n) = f(m, n) - Real.0
        f(m, n) - Real.0 = f(m, n)
        f(m, n) = pos_part_2(f, m, n) - neg_part_2(f, m, n)
    }
}

/// The absolute value equals positive part plus negative part.
theorem abs_eq_pos_plus_neg(f: (Nat, Nat) -> Real, m: Nat, n: Nat) {
    abs_fn_2(f, m, n) = pos_part_2(f, m, n) + neg_part_2(f, m, n)
} by {
    abs_fn_2(f, m, n) = f(m, n).abs
    pos_part_2(f, m, n) = f(m, n).max(Real.0)
    neg_part_2(f, m, n) = (-f(m, n)).max(Real.0)
    if f(m, n).is_negative {
        f(m, n).abs = -f(m, n)
        f(m, n).max(Real.0) = Real.0
        pos_part_2(f, m, n) = Real.0
        (-f(m, n)).max(Real.0) = -f(m, n)
        neg_part_2(f, m, n) = -f(m, n)
        pos_part_2(f, m, n) + neg_part_2(f, m, n) = Real.0 + (-f(m, n))
        Real.0 + (-f(m, n)) = -f(m, n)
        abs_fn_2(f, m, n) = pos_part_2(f, m, n) + neg_part_2(f, m, n)
    } else {
        f(m, n).abs = f(m, n)
        f(m, n).max(Real.0) = f(m, n)
        pos_part_2(f, m, n) = f(m, n)
        (-f(m, n)).max(Real.0) = Real.0
        neg_part_2(f, m, n) = Real.0
        pos_part_2(f, m, n) + neg_part_2(f, m, n) = f(m, n) + Real.0
        f(m, n) + Real.0 = f(m, n)
        abs_fn_2(f, m, n) = pos_part_2(f, m, n) + neg_part_2(f, m, n)
    }
}

/// The positive part is bounded by the absolute value.
theorem pos_part_le_abs(f: (Nat, Nat) -> Real, m: Nat, n: Nat) {
    pos_part_2(f, m, n) <= abs_fn_2(f, m, n)
} by {
    abs_fn_2(f, m, n) = pos_part_2(f, m, n) + neg_part_2(f, m, n)
    neg_part_2(f, m, n) >= Real.0
    pos_part_2(f, m, n) <= pos_part_2(f, m, n) + neg_part_2(f, m, n)
    pos_part_2(f, m, n) <= abs_fn_2(f, m, n)
}

/// The negative part is bounded by the absolute value.
theorem neg_part_le_abs(f: (Nat, Nat) -> Real, m: Nat, n: Nat) {
    neg_part_2(f, m, n) <= abs_fn_2(f, m, n)
} by {
    abs_fn_2(f, m, n) = pos_part_2(f, m, n) + neg_part_2(f, m, n)
    pos_part_2(f, m, n) >= Real.0
    neg_part_2(f, m, n) <= pos_part_2(f, m, n) + neg_part_2(f, m, n)
    neg_part_2(f, m, n) <= abs_fn_2(f, m, n)
}

/// The positive part is pointwise less than or equal to the absolute value.
theorem pos_part_lte_abs_fn(f: (Nat, Nat) -> Real) {
    lte_fn_2(pos_part_2(f), abs_fn_2(f))
} by {
    forall(m: Nat, n: Nat) {
        pos_part_2(f, m, n) <= abs_fn_2(f, m, n)
    }
}

/// The negative part is pointwise less than or equal to the absolute value.
theorem neg_part_lte_abs_fn(f: (Nat, Nat) -> Real) {
    lte_fn_2(neg_part_2(f), abs_fn_2(f))
} by {
    forall(m: Nat, n: Nat) {
        neg_part_2(f, m, n) <= abs_fn_2(f, m, n)
    }
}

/// Rectangular sum of positive part is bounded by rectangular sum of absolute value.
theorem rectangular_sum_pos_part_le_abs(f: (Nat, Nat) -> Real, m: Nat, n: Nat) {
    rectangular_sum(pos_part_2(f), m, n) <= rectangular_sum(abs_fn_2(f), m, n)
} by {
    nonneg_fn_2(pos_part_2(f))
    forall(i: Nat, j: Nat) {
        abs_fn_2(f, i, j) = f(i, j).abs
        f(i, j).abs >= Real.0
        abs_fn_2(f, i, j) >= Real.0
    }
    nonneg_fn_2(abs_fn_2(f))
    lte_fn_2(pos_part_2(f), abs_fn_2(f))
}

/// Rectangular sum of negative part is bounded by rectangular sum of absolute value.
theorem rectangular_sum_neg_part_le_abs(f: (Nat, Nat) -> Real, m: Nat, n: Nat) {
    rectangular_sum(neg_part_2(f), m, n) <= rectangular_sum(abs_fn_2(f), m, n)
} by {
    nonneg_fn_2(neg_part_2(f))
    forall(i: Nat, j: Nat) {
        abs_fn_2(f, i, j) = f(i, j).abs
        f(i, j).abs >= Real.0
        abs_fn_2(f, i, j) >= Real.0
    }
    nonneg_fn_2(abs_fn_2(f))
    lte_fn_2(neg_part_2(f), abs_fn_2(f))
}

/// A function equals the difference of its positive and negative parts.
theorem fn_eq_sub_pos_neg(f: (Nat, Nat) -> Real) {
    f = sub_fn_2(pos_part_2(f), neg_part_2(f))
} by {
    forall(m: Nat, n: Nat) {
        sub_fn_2(pos_part_2(f), neg_part_2(f), m, n) = pos_part_2(f, m, n) - neg_part_2(f, m, n)
        f(m, n) = pos_part_2(f, m, n) - neg_part_2(f, m, n)
        f(m, n) = sub_fn_2(pos_part_2(f), neg_part_2(f), m, n)
    }
}

/// Rectangular sum of f equals difference of rectangular sums of positive and negative parts.
theorem rectangular_sum_decomposition(f: (Nat, Nat) -> Real, m: Nat, n: Nat) {
    rectangular_sum(f, m, n) = rectangular_sum(pos_part_2(f), m, n) - rectangular_sum(neg_part_2(f), m, n)
} by {
    f = sub_fn_2(pos_part_2(f), neg_part_2(f))
    rectangular_sum(f, m, n) = rectangular_sum(sub_fn_2(pos_part_2(f), neg_part_2(f)), m, n)
    rectangular_sum(sub_fn_2(pos_part_2(f), neg_part_2(f)), m, n) = rectangular_sum(pos_part_2(f), m, n) - rectangular_sum(neg_part_2(f), m, n)
    rectangular_sum(f, m, n) = rectangular_sum(pos_part_2(f), m, n) - rectangular_sum(neg_part_2(f), m, n)
}

/// Subtraction as addition with negation for sequences.
theorem sub_seq_eq_add_neg(a: Nat -> Real, b: Nat -> Real) {
    sub_seq(a, b) = add_seq(a, neg_seq(b))
} by {
    forall(n: Nat) {
        sub_seq(a, b, n) = a(n) - b(n)
        neg_seq(b, n) = -b(n)
        add_seq(a, neg_seq(b), n) = a(n) + neg_seq(b, n)
        add_seq(a, neg_seq(b), n) = a(n) + (-b(n))
        a(n) + (-b(n)) = a(n) - b(n)
        sub_seq(a, b, n) = add_seq(a, neg_seq(b), n)
    }
}

/// The absolute value function is nonnegative.
theorem abs_fn_2_nonneg(f: (Nat, Nat) -> Real) {
    nonneg_fn_2(abs_fn_2(f))
} by {
    forall(m: Nat, n: Nat) {
        abs_fn_2(f, m, n) = f(m, n).abs
        f(m, n).abs >= Real.0
        abs_fn_2(f, m, n) >= Real.0
    }
}

/// Rectangular sums of absolute values are doubly increasing.
theorem rectangular_sum_abs_doubly_increasing(f: (Nat, Nat) -> Real) {
    doubly_increasing(rectangular_sum(abs_fn_2(f)))
} by {
    nonneg_fn_2(abs_fn_2(f))
    doubly_increasing(rectangular_sum(abs_fn_2(f)))
}

/// Rectangular sums of positive part are doubly increasing.
theorem rectangular_sum_pos_part_doubly_increasing(f: (Nat, Nat) -> Real) {
    doubly_increasing(rectangular_sum(pos_part_2(f)))
} by {
    nonneg_fn_2(pos_part_2(f))
    doubly_increasing(rectangular_sum(pos_part_2(f)))
}

/// Rectangular sums of negative part are doubly increasing.
theorem rectangular_sum_neg_part_doubly_increasing(f: (Nat, Nat) -> Real) {
    doubly_increasing(rectangular_sum(neg_part_2(f)))
} by {
    nonneg_fn_2(neg_part_2(f))
    doubly_increasing(rectangular_sum(neg_part_2(f)))
}

/// The supremum of abs serves as an upper bound for positive part rectangular sums.
theorem rectangular_sum_pos_part_bounded_by_abs_supremum(f: (Nat, Nat) -> Real, s: Real) {
    double_image_is_supremum(rectangular_sum(abs_fn_2(f)), s)
    implies
    double_is_upper_bound(rectangular_sum(pos_part_2(f)), s)
} by {
    if double_image_is_supremum(rectangular_sum(abs_fn_2(f)), s) {
        double_is_upper_bound(rectangular_sum(abs_fn_2(f)), s)
        forall(m: Nat, n: Nat) {
            rectangular_sum(abs_fn_2(f), m, n) <= s
            rectangular_sum(pos_part_2(f), m, n) <= rectangular_sum(abs_fn_2(f), m, n)
            rectangular_sum(pos_part_2(f), m, n) <= s
        }
        double_is_upper_bound(rectangular_sum(pos_part_2(f)), s)
    }
}

/// The supremum of abs serves as an upper bound for negative part rectangular sums.
theorem rectangular_sum_neg_part_bounded_by_abs_supremum(f: (Nat, Nat) -> Real, s: Real) {
    double_image_is_supremum(rectangular_sum(abs_fn_2(f)), s)
    implies
    double_is_upper_bound(rectangular_sum(neg_part_2(f)), s)
} by {
    if double_image_is_supremum(rectangular_sum(abs_fn_2(f)), s) {
        double_is_upper_bound(rectangular_sum(abs_fn_2(f)), s)
        forall(m: Nat, n: Nat) {
            rectangular_sum(abs_fn_2(f), m, n) <= s
            rectangular_sum(neg_part_2(f), m, n) <= rectangular_sum(abs_fn_2(f), m, n)
            rectangular_sum(neg_part_2(f), m, n) <= s
        }
        double_is_upper_bound(rectangular_sum(neg_part_2(f)), s)
    }
}

/// Convert the double image to a set.
define double_image_set(f: (Nat, Nat) -> Real) -> Set[Real] {
    Set[Real].new(double_image(f))
}

theorem double_image_set_nonempty(f: (Nat, Nat) -> Real) {
    is_nonempty(double_image_set(f))
} by {
    double_image(f, f(Nat.0, Nat.0))
    double_image_set(f).contains(f(Nat.0, Nat.0))
    is_nonempty(double_image_set(f))
}

theorem double_image_set_upper_bound_of_double_is_upper_bound(f: (Nat, Nat) -> Real, b: Real) {
    double_is_upper_bound(f, b) implies is_set_upper_bound(double_image_set(f), b)
} by {
    if double_is_upper_bound(f, b) {
        forall(x: Real) {
            if double_image_set(f).contains(x) {
                double_image(f, x)
                let (m: Nat, n: Nat) satisfy {
                    f(m, n) = x
                }
                f(m, n) <= b
                x <= b
            }
        }
    }
}

/// The supremum of the double image as a set equals the supremum as a double function.
theorem double_image_supremum_equiv(f: (Nat, Nat) -> Real, s_val: Real) {
    is_set_supremum(double_image_set(f), s_val)
    implies
    double_image_is_supremum(f, s_val)
} by {
    if is_set_supremum(double_image_set(f), s_val) {
        // Part 1: s_val is an upper bound
        forall(m: Nat, n: Nat) {
            double_image(f, f(m, n))
            double_image_set(f).contains(f(m, n))
            f(m, n) <= s_val
        }
        double_is_upper_bound(f, s_val)

        // Part 2: s_val is the least upper bound
        forall(b: Real) {
            if b < s_val {
                // s_val is the least upper bound of the set, so b is not an upper bound
                not is_set_upper_bound(double_image_set(f), b)
                exists(x0: Real) {
                    double_image_set(f).contains(x0) and not x0 <= b
                }
                let x0: Real satisfy {
                    double_image_set(f).contains(x0) and not x0 <= b
                }
                x0 > b
                exists(x: Real) {
                    double_image_set(f).contains(x) and x > b
                }
                // This means there exists x in the image such that x > b
                let x: Real satisfy {
                    double_image_set(f).contains(x) and x > b
                }
                // x is in the double image, so x = f(m, n) for some m, n
                double_image(f, x)
                let (m: Nat, n: Nat) satisfy {
                    f(m, n) = x
                }
                f(m, n) > b
                not double_is_upper_bound(f, b)
            }
        }

        double_image_is_supremum(f, s_val)
    }
}

/// If the double sum of abs converges, then the double sum of pos_part converges.
theorem double_sum_pos_part_converges_of_supremum(f: (Nat, Nat) -> Real, s: Real) {
    double_image_is_supremum(rectangular_sum(abs_fn_2(f)), s)
    implies
    double_sum_converges(pos_part_2(f))
} by {
    if double_image_is_supremum(rectangular_sum(abs_fn_2(f)), s) {
        double_is_upper_bound(rectangular_sum(pos_part_2(f)), s)

        doubly_increasing(rectangular_sum(pos_part_2(f)))

        let img_set = double_image_set(rectangular_sum(pos_part_2(f)))
        double_image_set_nonempty(rectangular_sum(pos_part_2(f)))
        is_nonempty(img_set)

        double_image_set_upper_bound_of_double_is_upper_bound(rectangular_sum(pos_part_2(f)), s)
        is_set_upper_bound(double_image_set(rectangular_sum(pos_part_2(f))), s)
        is_set_upper_bound(img_set, s)
        has_upper_bound(img_set)

        // By completeness, the supremum exists
        let s_pos: Real satisfy {
            is_set_supremum(img_set, s_pos)
        }

        // Convert to double_image_is_supremum
        double_image_is_supremum(rectangular_sum(pos_part_2(f)), s_pos)

        double_converges_to(rectangular_sum(pos_part_2(f)), s_pos)
        double_converges(rectangular_sum(pos_part_2(f)))
        double_sum_converges(pos_part_2(f))
    }
}

/// If the double sum of abs converges, then the double sum of neg_part converges.
theorem double_sum_neg_part_converges_of_supremum(f: (Nat, Nat) -> Real, s: Real) {
    double_image_is_supremum(rectangular_sum(abs_fn_2(f)), s)
    implies
    double_sum_converges(neg_part_2(f))
} by {
    if double_image_is_supremum(rectangular_sum(abs_fn_2(f)), s) {
        double_is_upper_bound(rectangular_sum(neg_part_2(f)), s)

        doubly_increasing(rectangular_sum(neg_part_2(f)))

        let img_set = double_image_set(rectangular_sum(neg_part_2(f)))
        double_image_set_nonempty(rectangular_sum(neg_part_2(f)))
        is_nonempty(img_set)

        double_image_set_upper_bound_of_double_is_upper_bound(rectangular_sum(neg_part_2(f)), s)
        is_set_upper_bound(double_image_set(rectangular_sum(neg_part_2(f))), s)
        is_set_upper_bound(img_set, s)
        has_upper_bound(img_set)

        // By completeness, the supremum exists
        let s_neg: Real satisfy {
            is_set_supremum(img_set, s_neg)
        }

        // Convert to double_image_is_supremum
        double_image_is_supremum(rectangular_sum(neg_part_2(f)), s_neg)

        double_converges_to(rectangular_sum(neg_part_2(f)), s_neg)
        double_converges(rectangular_sum(neg_part_2(f)))
        double_sum_converges(neg_part_2(f))
    }
}

/// The limit of a difference of sequences equals the difference of their limits.
theorem limit_sub_seq(a: Nat -> Real, b: Nat -> Real) {
    converges(a) and converges(b)
    implies
    limit(sub_seq(a, b)) = limit(a) - limit(b)
} by {
    if converges(a) and converges(b) {
        sub_seq(a, b) = add_seq(a, neg_seq(b))

        converges(neg_seq(b))

        converges_to(add_seq(a, neg_seq(b)), limit(a) + limit(neg_seq(b)))
        limit(add_seq(a, neg_seq(b))) = limit(a) + limit(neg_seq(b))

        converges_to(neg_seq(b), -limit(b))
        converges_to(neg_seq(b), limit(neg_seq(b)))
        limit(neg_seq(b)) = -limit(b)

        limit(add_seq(a, neg_seq(b))) = limit(a) + (-limit(b))
        limit(a) + (-limit(b)) = limit(a) - limit(b)
        limit(sub_seq(a, b)) = limit(a) - limit(b)
    }
}

/// Partial sums of a row can be decomposed into positive and negative parts.
theorem row_partial_decomposition(f: (Nat, Nat) -> Real, i: Nat, n: Nat) {
    partial(f(i), n) = partial(pos_part_2(f)(i), n) - partial(neg_part_2(f)(i), n)
} by {
    define p(k: Nat) -> Bool {
        partial(f(i), k) = partial(pos_part_2(f)(i), k) - partial(neg_part_2(f)(i), k)
    }

    partial(f(i), Nat.0) = Real.0
    partial(pos_part_2(f)(i), Nat.0) = Real.0
    partial(neg_part_2(f)(i), Nat.0) = Real.0
    Real.0 - Real.0 = Real.0
    p(Nat.0)

    forall(k: Nat) {
        if p(k) {
            partial(f(i), k.suc) = partial(f(i), k) + f(i, k)
            partial(pos_part_2(f)(i), k.suc) = partial(pos_part_2(f)(i), k) + pos_part_2(f)(i, k)
            partial(neg_part_2(f)(i), k.suc) = partial(neg_part_2(f)(i), k) + neg_part_2(f)(i, k)

            pos_part_2(f)(i, k) = pos_part_2(f, i, k)
            neg_part_2(f)(i, k) = neg_part_2(f, i, k)

            f(i, k) = pos_part_2(f, i, k) - neg_part_2(f, i, k)

            partial(f(i), k) = partial(pos_part_2(f)(i), k) - partial(neg_part_2(f)(i), k)
            partial(f(i), k) + f(i, k) = (partial(pos_part_2(f)(i), k) - partial(neg_part_2(f)(i), k)) + (pos_part_2(f, i, k) - neg_part_2(f, i, k))
            let aa = partial(pos_part_2(f)(i), k)
            let bb = partial(neg_part_2(f)(i), k)
            let cc = pos_part_2(f, i, k)
            let dd = neg_part_2(f, i, k)
            (aa + cc) - (bb + dd) = aa + cc - bb - dd
            aa + cc - bb = aa - bb + cc
            cc + (aa - bb) - dd = cc - dd + (aa - bb)
            (aa - bb) + (cc - dd) = cc - dd + (aa - bb)
            (aa - bb) + cc = cc + (aa - bb)
            (partial(pos_part_2(f)(i), k) - partial(neg_part_2(f)(i), k)) + (pos_part_2(f, i, k) - neg_part_2(f, i, k)) = (partial(pos_part_2(f)(i), k) + pos_part_2(f, i, k)) - (partial(neg_part_2(f)(i), k) + neg_part_2(f, i, k))
            partial(f(i), k.suc) = (partial(pos_part_2(f)(i), k) + pos_part_2(f, i, k)) - (partial(neg_part_2(f)(i), k) + neg_part_2(f, i, k))
            partial(f(i), k.suc) = partial(pos_part_2(f)(i), k.suc) - partial(neg_part_2(f)(i), k.suc)
            p(k.suc)
        }
    }

    p(n)
}

/// Row infinite sum can be decomposed into positive and negative parts.
theorem row_infinite_sum_decomposition(f: (Nat, Nat) -> Real, i: Nat) {
    converges(partial(pos_part_2(f)(i))) and converges(partial(neg_part_2(f)(i)))
    implies
    row_infinite_sum(f, i) = row_infinite_sum(pos_part_2(f), i) - row_infinite_sum(neg_part_2(f), i)
} by {
    if converges(partial(pos_part_2(f)(i))) and converges(partial(neg_part_2(f)(i))) {
        forall(n: Nat) {
            partial(f(i), n) = partial(pos_part_2(f)(i), n) - partial(neg_part_2(f)(i), n)
            sub_seq(partial(pos_part_2(f)(i)), partial(neg_part_2(f)(i)), n) = partial(pos_part_2(f)(i), n) - partial(neg_part_2(f)(i), n)
            partial(f(i), n) = sub_seq(partial(pos_part_2(f)(i)), partial(neg_part_2(f)(i)), n)
        }
        partial(f(i)) = sub_seq(partial(pos_part_2(f)(i)), partial(neg_part_2(f)(i)))

        limit(sub_seq(partial(pos_part_2(f)(i)), partial(neg_part_2(f)(i)))) = limit(partial(pos_part_2(f)(i))) - limit(partial(neg_part_2(f)(i)))
        limit(partial(f(i))) = limit(partial(pos_part_2(f)(i))) - limit(partial(neg_part_2(f)(i)))

        row_infinite_sum(f, i) = limit(partial(f(i)))
        row_infinite_sum(pos_part_2(f), i) = limit(partial(pos_part_2(f)(i)))
        row_infinite_sum(neg_part_2(f), i) = limit(partial(neg_part_2(f)(i)))

        row_infinite_sum(f, i) = row_infinite_sum(pos_part_2(f), i) - row_infinite_sum(neg_part_2(f), i)
    }
}

/// Row sums sequence decomposes into positive and negative parts.
theorem row_sums_seq_decomposition(f: (Nat, Nat) -> Real, i: Nat) {
    converges(partial(pos_part_2(f)(i))) and converges(partial(neg_part_2(f)(i)))
    implies
    row_sums_seq(f, i) = row_sums_seq(pos_part_2(f), i) - row_sums_seq(neg_part_2(f), i)
} by {
    if converges(partial(pos_part_2(f)(i))) and converges(partial(neg_part_2(f)(i))) {
        row_sums_seq(f, i) = row_infinite_sum(f, i)
        row_sums_seq(pos_part_2(f), i) = row_infinite_sum(pos_part_2(f), i)
        row_sums_seq(neg_part_2(f), i) = row_infinite_sum(neg_part_2(f), i)

        row_infinite_sum(f, i) = row_infinite_sum(pos_part_2(f), i) - row_infinite_sum(neg_part_2(f), i)
        row_sums_seq(f, i) = row_sums_seq(pos_part_2(f), i) - row_sums_seq(neg_part_2(f), i)
    }
}

/// Partial sums of a column can be decomposed into positive and negative parts.
theorem col_partial_decomposition(f: (Nat, Nat) -> Real, j: Nat, m: Nat) {
    partial(flip(f, j), m) = partial(flip(pos_part_2(f), j), m) - partial(flip(neg_part_2(f), j), m)
} by {
    define p(k: Nat) -> Bool {
        partial(flip(f, j), k) = partial(flip(pos_part_2(f), j), k) - partial(flip(neg_part_2(f), j), k)
    }

    partial(flip(f, j), Nat.0) = Real.0
    partial(flip(pos_part_2(f), j), Nat.0) = Real.0
    partial(flip(neg_part_2(f), j), Nat.0) = Real.0
    Real.0 - Real.0 = Real.0
    p(Nat.0)

    forall(k: Nat) {
        if p(k) {
            partial(flip(f, j), k.suc) = partial(flip(f, j), k) + flip(f, j, k)
            partial(flip(pos_part_2(f), j), k.suc) = partial(flip(pos_part_2(f), j), k) + flip(pos_part_2(f), j, k)
            partial(flip(neg_part_2(f), j), k.suc) = partial(flip(neg_part_2(f), j), k) + flip(neg_part_2(f), j, k)

            flip(f, j, k) = f(k, j)
            flip(pos_part_2(f), j, k) = pos_part_2(f, k, j)
            flip(neg_part_2(f), j, k) = neg_part_2(f, k, j)

            f(k, j) = pos_part_2(f, k, j) - neg_part_2(f, k, j)

            partial(flip(f, j), k) = partial(flip(pos_part_2(f), j), k) - partial(flip(neg_part_2(f), j), k)
            partial(flip(f, j), k) + flip(f, j, k) = (partial(flip(pos_part_2(f), j), k) - partial(flip(neg_part_2(f), j), k)) + (pos_part_2(f, k, j) - neg_part_2(f, k, j))
            let aa = partial(flip(pos_part_2(f), j), k)
            let bb = partial(flip(neg_part_2(f), j), k)
            let cc = pos_part_2(f, k, j)
            let dd = neg_part_2(f, k, j)
            (aa + cc) - (bb + dd) = aa + cc - bb - dd
            aa + cc - bb = aa - bb + cc
            cc + (aa - bb) - dd = cc - dd + (aa - bb)
            (aa - bb) + (cc - dd) = cc - dd + (aa - bb)
            (aa - bb) + cc = cc + (aa - bb)
            (partial(flip(pos_part_2(f), j), k) - partial(flip(neg_part_2(f), j), k)) + (pos_part_2(f, k, j) - neg_part_2(f, k, j)) = (partial(flip(pos_part_2(f), j), k) + pos_part_2(f, k, j)) - (partial(flip(neg_part_2(f), j), k) + neg_part_2(f, k, j))
            partial(flip(f, j), k.suc) = (partial(flip(pos_part_2(f), j), k) + flip(pos_part_2(f), j, k)) - (partial(flip(neg_part_2(f), j), k) + flip(neg_part_2(f), j, k))
            partial(flip(f, j), k.suc) = partial(flip(pos_part_2(f), j), k.suc) - partial(flip(neg_part_2(f), j), k.suc)
            p(k.suc)
        }
    }

    p(m)
}

/// Column infinite sum can be decomposed into positive and negative parts.
theorem col_infinite_sum_decomposition(f: (Nat, Nat) -> Real, j: Nat) {
    converges(partial(flip(pos_part_2(f), j))) and converges(partial(flip(neg_part_2(f), j)))
    implies
    col_infinite_sum(f, j) = col_infinite_sum(pos_part_2(f), j) - col_infinite_sum(neg_part_2(f), j)
} by {
    if converges(partial(flip(pos_part_2(f), j))) and converges(partial(flip(neg_part_2(f), j))) {
        forall(m: Nat) {
            partial(flip(f, j), m) = partial(flip(pos_part_2(f), j), m) - partial(flip(neg_part_2(f), j), m)
            sub_seq(partial(flip(pos_part_2(f), j)), partial(flip(neg_part_2(f), j)), m) = partial(flip(pos_part_2(f), j), m) - partial(flip(neg_part_2(f), j), m)
            partial(flip(f, j), m) = sub_seq(partial(flip(pos_part_2(f), j)), partial(flip(neg_part_2(f), j)), m)
        }
        partial(flip(f, j)) = sub_seq(partial(flip(pos_part_2(f), j)), partial(flip(neg_part_2(f), j)))

        limit(sub_seq(partial(flip(pos_part_2(f), j)), partial(flip(neg_part_2(f), j)))) = limit(partial(flip(pos_part_2(f), j))) - limit(partial(flip(neg_part_2(f), j)))
        limit(partial(flip(f, j))) = limit(partial(flip(pos_part_2(f), j))) - limit(partial(flip(neg_part_2(f), j)))

        col_infinite_sum(f, j) = limit(partial(flip(f, j)))
        col_infinite_sum(pos_part_2(f), j) = limit(partial(flip(pos_part_2(f), j)))
        col_infinite_sum(neg_part_2(f), j) = limit(partial(flip(neg_part_2(f), j)))

        col_infinite_sum(f, j) = col_infinite_sum(pos_part_2(f), j) - col_infinite_sum(neg_part_2(f), j)
    }
}

/// Column sums sequence decomposes into positive and negative parts.
theorem col_sums_seq_decomposition(f: (Nat, Nat) -> Real, j: Nat) {
    converges(partial(flip(pos_part_2(f), j))) and converges(partial(flip(neg_part_2(f), j)))
    implies
    col_sums_seq(f, j) = col_sums_seq(pos_part_2(f), j) - col_sums_seq(neg_part_2(f), j)
} by {
    if converges(partial(flip(pos_part_2(f), j))) and converges(partial(flip(neg_part_2(f), j))) {
        col_sums_seq(f, j) = col_infinite_sum(f, j)
        col_sums_seq(pos_part_2(f), j) = col_infinite_sum(pos_part_2(f), j)
        col_sums_seq(neg_part_2(f), j) = col_infinite_sum(neg_part_2(f), j)

        col_infinite_sum(f, j) = col_infinite_sum(pos_part_2(f), j) - col_infinite_sum(neg_part_2(f), j)
        col_sums_seq(f, j) = col_sums_seq(pos_part_2(f), j) - col_sums_seq(neg_part_2(f), j)
    }
}

/// The partial sums of a difference equal the difference of partial sums.
theorem partial_sub_seq(a: Nat -> Real, b: Nat -> Real, n: Nat) {
    partial(sub_seq(a, b), n) = partial(a, n) - partial(b, n)
} by {
    define p(k: Nat) -> Bool {
        partial(sub_seq(a, b), k) = partial(a, k) - partial(b, k)
    }

    partial(sub_seq(a, b), Nat.0) = Real.0
    partial(a, Nat.0) = Real.0
    partial(b, Nat.0) = Real.0
    Real.0 - Real.0 = Real.0
    p(Nat.0)

    forall(k: Nat) {
        if p(k) {
            partial(sub_seq(a, b), k.suc) = partial(sub_seq(a, b), k) + sub_seq(a, b, k)
            sub_seq(a, b, k) = a(k) - b(k)
            partial(sub_seq(a, b), k) = partial(a, k) - partial(b, k)
            partial(sub_seq(a, b), k) + sub_seq(a, b, k) = (partial(a, k) - partial(b, k)) + (a(k) - b(k))
            (partial(a, k) - partial(b, k)) + (a(k) - b(k)) = (partial(a, k) + a(k)) - (partial(b, k) + b(k))
            partial(a, k.suc) = partial(a, k) + a(k)
            partial(b, k.suc) = partial(b, k) + b(k)
            partial(sub_seq(a, b), k.suc) = partial(a, k.suc) - partial(b, k.suc)
            p(k.suc)
        }
    }

    p(n)
}

// Note: The following helper theorems for Fubini assume (but do not prove) that
// double_sum(f) = double_sum(pos_part_2(f)) - double_sum(neg_part_2(f))
// This linearity property requires a theorem about double limits and subtraction.

/// For absolutely convergent sums, the row-iterated limit equals the double sum.
/// This is a helper lemma for Fubini's theorem.
theorem row_iterated_limit_eq_double_sum(f: (Nat, Nat) -> Real, s: Real, s_pos: Real, s_neg: Real) {
    double_image_is_supremum(rectangular_sum(abs_fn_2(f)), s) and
    double_image_is_supremum(rectangular_sum(pos_part_2(f)), s_pos) and
    double_image_is_supremum(rectangular_sum(neg_part_2(f)), s_neg)
    implies
    double_sum(f) = limit(partial(row_sums_seq(f)))
} by {
    if double_image_is_supremum(rectangular_sum(abs_fn_2(f)), s) and
       double_image_is_supremum(rectangular_sum(pos_part_2(f)), s_pos) and
       double_image_is_supremum(rectangular_sum(neg_part_2(f)), s_neg) {

        // Apply Tonelli to get the limits for pos and neg parts
        nonneg_fn_2(pos_part_2(f))
        double_sum(pos_part_2(f)) = limit(partial(row_sums_seq(pos_part_2(f))))

        nonneg_fn_2(neg_part_2(f))
        double_sum(neg_part_2(f)) = limit(partial(row_sums_seq(neg_part_2(f))))

        // Row sums decompose
        all_rows_converge(pos_part_2(f))
        all_rows_converge(neg_part_2(f))

        forall(i: Nat) {
            converges(partial(pos_part_2(f)(i)))
            converges(partial(neg_part_2(f)(i)))
            row_sums_seq(f, i) = row_sums_seq(pos_part_2(f), i) - row_sums_seq(neg_part_2(f), i)
            sub_seq(row_sums_seq(pos_part_2(f)), row_sums_seq(neg_part_2(f)), i) =
                row_sums_seq(pos_part_2(f), i) - row_sums_seq(neg_part_2(f), i)
            row_sums_seq(f, i) =
                sub_seq(row_sums_seq(pos_part_2(f)), row_sums_seq(neg_part_2(f)), i)
        }
        row_sums_seq(f) = sub_seq(row_sums_seq(pos_part_2(f)), row_sums_seq(neg_part_2(f)))

        // Establish that partial(row_sums_seq) equals row_limit(rectangular_sum)
        row_limit(rectangular_sum(pos_part_2(f))) = partial(row_sums_seq(pos_part_2(f)))

        row_limit(rectangular_sum(neg_part_2(f))) = partial(row_sums_seq(neg_part_2(f)))

        // Establish doubly_increasing and upper bounds
        doubly_increasing(rectangular_sum(pos_part_2(f)))
        doubly_increasing(rectangular_sum(neg_part_2(f)))

        double_is_upper_bound(rectangular_sum(pos_part_2(f)), s_pos)
        double_is_upper_bound(rectangular_sum(neg_part_2(f)), s_neg)

        // Prove convergence of row_limit sequences
        converges(row_limit(rectangular_sum(pos_part_2(f))))
        converges(row_limit(rectangular_sum(neg_part_2(f))))

        // Therefore partial(row_sums_seq) converges
        converges(partial(row_sums_seq(pos_part_2(f))))
        converges(partial(row_sums_seq(neg_part_2(f))))

        // Partial sums decompose
        forall(n: Nat) {
            partial(row_sums_seq(f), n) = partial(row_sums_seq(pos_part_2(f)), n) - partial(row_sums_seq(neg_part_2(f)), n)
            sub_seq(partial(row_sums_seq(pos_part_2(f))), partial(row_sums_seq(neg_part_2(f))), n) =
                partial(row_sums_seq(pos_part_2(f)), n) - partial(row_sums_seq(neg_part_2(f)), n)
            partial(row_sums_seq(f), n) =
                sub_seq(partial(row_sums_seq(pos_part_2(f))), partial(row_sums_seq(neg_part_2(f))), n)
        }
        partial(row_sums_seq(f)) = sub_seq(partial(row_sums_seq(pos_part_2(f))), partial(row_sums_seq(neg_part_2(f))))

        // Apply limit_sub_seq
        limit(sub_seq(partial(row_sums_seq(pos_part_2(f))), partial(row_sums_seq(neg_part_2(f))))) =
            limit(partial(row_sums_seq(pos_part_2(f)))) - limit(partial(row_sums_seq(neg_part_2(f))))
        limit(partial(row_sums_seq(f))) = limit(partial(row_sums_seq(pos_part_2(f)))) - limit(partial(row_sums_seq(neg_part_2(f))))
        limit(partial(row_sums_seq(f))) = double_sum(pos_part_2(f)) - double_sum(neg_part_2(f))

        // Prove the decomposition using double_limit_sub
        // First show that rectangular_sum(f) = sub_fn_2(rectangular_sum(pos_part_2(f)), rectangular_sum(neg_part_2(f)))
        forall(m: Nat, n: Nat) {
            rectangular_sum(f, m, n) = rectangular_sum(pos_part_2(f), m, n) - rectangular_sum(neg_part_2(f), m, n)
            sub_fn_2(rectangular_sum(pos_part_2(f)), rectangular_sum(neg_part_2(f)), m, n) =
                rectangular_sum(pos_part_2(f), m, n) - rectangular_sum(neg_part_2(f), m, n)
            rectangular_sum(f, m, n) = sub_fn_2(rectangular_sum(pos_part_2(f)), rectangular_sum(neg_part_2(f)), m, n)
        }
        rectangular_sum(f) = sub_fn_2(rectangular_sum(pos_part_2(f)), rectangular_sum(neg_part_2(f)))

        // Both pos and neg parts converge
        double_sum_converges(pos_part_2(f))
        double_converges(rectangular_sum(pos_part_2(f)))
        double_sum_converges(neg_part_2(f))
        double_converges(rectangular_sum(neg_part_2(f)))

        // Apply double_limit_sub
        double_converges_to(sub_fn_2(rectangular_sum(pos_part_2(f)), rectangular_sum(neg_part_2(f))),
            double_limit(rectangular_sum(pos_part_2(f))) - double_limit(rectangular_sum(neg_part_2(f))))
        double_converges_to(rectangular_sum(f),
            double_limit(rectangular_sum(pos_part_2(f))) - double_limit(rectangular_sum(neg_part_2(f))))

        // Extract the equality
        double_limit(rectangular_sum(f)) = double_limit(rectangular_sum(pos_part_2(f))) - double_limit(rectangular_sum(neg_part_2(f)))
        double_sum(f) = double_sum(pos_part_2(f)) - double_sum(neg_part_2(f))
        double_sum(f) = limit(partial(row_sums_seq(f)))
    }
}

/// For absolutely convergent sums, the column-iterated limit equals the double sum.
/// This is a helper lemma for Fubini's theorem.
theorem col_iterated_limit_eq_double_sum(f: (Nat, Nat) -> Real, s: Real, s_pos: Real, s_neg: Real) {
    double_image_is_supremum(rectangular_sum(abs_fn_2(f)), s) and
    double_image_is_supremum(rectangular_sum(pos_part_2(f)), s_pos) and
    double_image_is_supremum(rectangular_sum(neg_part_2(f)), s_neg)
    implies
    double_sum(f) = limit(partial(col_sums_seq(f)))
} by {
    if double_image_is_supremum(rectangular_sum(abs_fn_2(f)), s) and
       double_image_is_supremum(rectangular_sum(pos_part_2(f)), s_pos) and
       double_image_is_supremum(rectangular_sum(neg_part_2(f)), s_neg) {

        // Apply Tonelli to get the limits for pos and neg parts
        nonneg_fn_2(pos_part_2(f))
        double_sum(pos_part_2(f)) = limit(partial(col_sums_seq(pos_part_2(f))))

        nonneg_fn_2(neg_part_2(f))
        double_sum(neg_part_2(f)) = limit(partial(col_sums_seq(neg_part_2(f))))

        // Column sums decompose
        all_cols_converge(pos_part_2(f))
        all_cols_converge(neg_part_2(f))

        forall(j: Nat) {
            converges(partial(flip(pos_part_2(f), j)))
            converges(partial(flip(neg_part_2(f), j)))
            col_sums_seq(f, j) = col_sums_seq(pos_part_2(f), j) - col_sums_seq(neg_part_2(f), j)
            sub_seq(col_sums_seq(pos_part_2(f)), col_sums_seq(neg_part_2(f)), j) =
                col_sums_seq(pos_part_2(f), j) - col_sums_seq(neg_part_2(f), j)
            col_sums_seq(f, j) =
                sub_seq(col_sums_seq(pos_part_2(f)), col_sums_seq(neg_part_2(f)), j)
        }
        col_sums_seq(f) = sub_seq(col_sums_seq(pos_part_2(f)), col_sums_seq(neg_part_2(f)))

        // Establish that partial(col_sums_seq) equals col_limit(rectangular_sum)
        col_limit(rectangular_sum(pos_part_2(f))) = partial(col_sums_seq(pos_part_2(f)))

        col_limit(rectangular_sum(neg_part_2(f))) = partial(col_sums_seq(neg_part_2(f)))

        // Establish doubly_increasing and upper bounds
        doubly_increasing(rectangular_sum(pos_part_2(f)))
        doubly_increasing(rectangular_sum(neg_part_2(f)))

        double_is_upper_bound(rectangular_sum(pos_part_2(f)), s_pos)
        double_is_upper_bound(rectangular_sum(neg_part_2(f)), s_neg)

        // Prove convergence of col_limit sequences
        converges(col_limit(rectangular_sum(pos_part_2(f))))
        converges(col_limit(rectangular_sum(neg_part_2(f))))

        // Therefore partial(col_sums_seq) converges
        converges(partial(col_sums_seq(pos_part_2(f))))
        converges(partial(col_sums_seq(neg_part_2(f))))

        // Partial sums decompose
        forall(n: Nat) {
            partial(col_sums_seq(f), n) = partial(col_sums_seq(pos_part_2(f)), n) - partial(col_sums_seq(neg_part_2(f)), n)
            sub_seq(partial(col_sums_seq(pos_part_2(f))), partial(col_sums_seq(neg_part_2(f))), n) =
                partial(col_sums_seq(pos_part_2(f)), n) - partial(col_sums_seq(neg_part_2(f)), n)
            partial(col_sums_seq(f), n) =
                sub_seq(partial(col_sums_seq(pos_part_2(f))), partial(col_sums_seq(neg_part_2(f))), n)
        }
        partial(col_sums_seq(f)) = sub_seq(partial(col_sums_seq(pos_part_2(f))), partial(col_sums_seq(neg_part_2(f))))

        // Apply limit_sub_seq
        limit(sub_seq(partial(col_sums_seq(pos_part_2(f))), partial(col_sums_seq(neg_part_2(f))))) =
            limit(partial(col_sums_seq(pos_part_2(f)))) - limit(partial(col_sums_seq(neg_part_2(f))))
        limit(partial(col_sums_seq(f))) = limit(partial(col_sums_seq(pos_part_2(f)))) - limit(partial(col_sums_seq(neg_part_2(f))))
        limit(partial(col_sums_seq(f))) = double_sum(pos_part_2(f)) - double_sum(neg_part_2(f))

        // Prove the decomposition using double_limit_sub
        // First show that rectangular_sum(f) = sub_fn_2(rectangular_sum(pos_part_2(f)), rectangular_sum(neg_part_2(f)))
        forall(m: Nat, n: Nat) {
            rectangular_sum(f, m, n) = rectangular_sum(pos_part_2(f), m, n) - rectangular_sum(neg_part_2(f), m, n)
            sub_fn_2(rectangular_sum(pos_part_2(f)), rectangular_sum(neg_part_2(f)), m, n) =
                rectangular_sum(pos_part_2(f), m, n) - rectangular_sum(neg_part_2(f), m, n)
            rectangular_sum(f, m, n) = sub_fn_2(rectangular_sum(pos_part_2(f)), rectangular_sum(neg_part_2(f)), m, n)
        }
        rectangular_sum(f) = sub_fn_2(rectangular_sum(pos_part_2(f)), rectangular_sum(neg_part_2(f)))

        // Both pos and neg parts converge
        double_sum_converges(pos_part_2(f))
        double_converges(rectangular_sum(pos_part_2(f)))
        double_sum_converges(neg_part_2(f))
        double_converges(rectangular_sum(neg_part_2(f)))

        // Apply double_limit_sub
        double_converges_to(sub_fn_2(rectangular_sum(pos_part_2(f)), rectangular_sum(neg_part_2(f))),
            double_limit(rectangular_sum(pos_part_2(f))) - double_limit(rectangular_sum(neg_part_2(f))))
        double_converges_to(rectangular_sum(f),
            double_limit(rectangular_sum(pos_part_2(f))) - double_limit(rectangular_sum(neg_part_2(f))))

        // Extract the equality
        double_limit(rectangular_sum(f)) = double_limit(rectangular_sum(pos_part_2(f))) - double_limit(rectangular_sum(neg_part_2(f)))
        double_sum(f) = double_sum(pos_part_2(f)) - double_sum(neg_part_2(f))
        double_sum(f) = limit(partial(col_sums_seq(f)))
    }
}

/// Fubini's theorem for row-wise iterated sums.
/// If the absolute value double sum converges, then the double sum equals the row-iterated sum.
theorem fubini_rows(f: (Nat, Nat) -> Real, s: Real) {
    double_image_is_supremum(rectangular_sum(abs_fn_2(f)), s)
    implies
    double_sum(f) = limit(partial(row_sums_seq(f)))
} by {
    if double_image_is_supremum(rectangular_sum(abs_fn_2(f)), s) {
        // Positive and negative parts have suprema bounded by s
        double_is_upper_bound(rectangular_sum(pos_part_2(f)), s)
        double_is_upper_bound(rectangular_sum(neg_part_2(f)), s)

        // Positive and negative parts are doubly increasing
        doubly_increasing(rectangular_sum(pos_part_2(f)))
        doubly_increasing(rectangular_sum(neg_part_2(f)))

        // Get suprema for positive and negative parts
        let img_pos = double_image_set(rectangular_sum(pos_part_2(f)))
        double_image_set_nonempty(rectangular_sum(pos_part_2(f)))
        is_nonempty(img_pos)

        double_image_set_upper_bound_of_double_is_upper_bound(rectangular_sum(pos_part_2(f)), s)
        is_set_upper_bound(double_image_set(rectangular_sum(pos_part_2(f))), s)
        is_set_upper_bound(img_pos, s)
        has_upper_bound(img_pos)

        let s_pos: Real satisfy {
            is_set_supremum(img_pos, s_pos)
        }
        double_image_is_supremum(rectangular_sum(pos_part_2(f)), s_pos)

        let img_neg = double_image_set(rectangular_sum(neg_part_2(f)))
        double_image_set_nonempty(rectangular_sum(neg_part_2(f)))
        is_nonempty(img_neg)

        double_image_set_upper_bound_of_double_is_upper_bound(rectangular_sum(neg_part_2(f)), s)
        is_set_upper_bound(double_image_set(rectangular_sum(neg_part_2(f))), s)
        is_set_upper_bound(img_neg, s)
        has_upper_bound(img_neg)

        let s_neg: Real satisfy {
            is_set_supremum(img_neg, s_neg)
        }
        double_image_is_supremum(rectangular_sum(neg_part_2(f)), s_neg)

        // Apply the helper theorem
        double_sum(f) = limit(partial(row_sums_seq(f)))
    }
}

/// Fubini's theorem for column-wise iterated sums.
/// If the absolute value double sum converges, then the double sum equals the column-iterated sum.
theorem fubini_cols(f: (Nat, Nat) -> Real, s: Real) {
    double_image_is_supremum(rectangular_sum(abs_fn_2(f)), s)
    implies
    double_sum(f) = limit(partial(col_sums_seq(f)))
} by {
    if double_image_is_supremum(rectangular_sum(abs_fn_2(f)), s) {
        // Positive and negative parts have suprema bounded by s
        double_is_upper_bound(rectangular_sum(pos_part_2(f)), s)
        double_is_upper_bound(rectangular_sum(neg_part_2(f)), s)

        // Positive and negative parts are doubly increasing
        doubly_increasing(rectangular_sum(pos_part_2(f)))
        doubly_increasing(rectangular_sum(neg_part_2(f)))

        // Get suprema for positive and negative parts
        let img_pos = double_image_set(rectangular_sum(pos_part_2(f)))
        double_image_set_nonempty(rectangular_sum(pos_part_2(f)))
        is_nonempty(img_pos)

        double_image_set_upper_bound_of_double_is_upper_bound(rectangular_sum(pos_part_2(f)), s)
        is_set_upper_bound(double_image_set(rectangular_sum(pos_part_2(f))), s)
        is_set_upper_bound(img_pos, s)
        has_upper_bound(img_pos)

        let s_pos: Real satisfy {
            is_set_supremum(img_pos, s_pos)
        }
        double_image_is_supremum(rectangular_sum(pos_part_2(f)), s_pos)

        let img_neg = double_image_set(rectangular_sum(neg_part_2(f)))
        double_image_set_nonempty(rectangular_sum(neg_part_2(f)))
        is_nonempty(img_neg)

        double_image_set_upper_bound_of_double_is_upper_bound(rectangular_sum(neg_part_2(f)), s)
        is_set_upper_bound(double_image_set(rectangular_sum(neg_part_2(f))), s)
        is_set_upper_bound(img_neg, s)
        has_upper_bound(img_neg)

        let s_neg: Real satisfy {
            is_set_supremum(img_neg, s_neg)
        }
        double_image_is_supremum(rectangular_sum(neg_part_2(f)), s_neg)

        // Apply the helper theorem
        double_sum(f) = limit(partial(col_sums_seq(f)))
    }
}

/// Fubini's theorem: If a double sum is absolutely convergent, it equals
/// the iterated sum in each direction, and both iterated sums are equal.
theorem fubini(f: (Nat, Nat) -> Real, s: Real) {
    double_image_is_supremum(rectangular_sum(abs_fn_2(f)), s)
    implies
    double_sum(f) = limit(partial(row_sums_seq(f))) and
    double_sum(f) = limit(partial(col_sums_seq(f))) and
    limit(partial(row_sums_seq(f))) = limit(partial(col_sums_seq(f)))
} by {
    if double_image_is_supremum(rectangular_sum(abs_fn_2(f)), s) {
        double_sum(f) = limit(partial(row_sums_seq(f)))

        double_sum(f) = limit(partial(col_sums_seq(f)))

        limit(partial(row_sums_seq(f))) = limit(partial(col_sums_seq(f)))
    }
}

/// When a double sum converges, the sequence of square sums converges to the double sum.
theorem square_sum_converges_to_double_sum(f: (Nat, Nat) -> Real) {
    double_sum_converges(f)
    implies
    converges_to(square_sum(f), double_sum(f))
} by {
    if double_sum_converges(f) {
        // double_sum_converges(f) means double_converges(rectangular_sum(f))
        double_converges(rectangular_sum(f))

        // This means there exists a value a such that double_converges_to(rectangular_sum(f), a)
        let a: Real satisfy {
            double_converges_to(rectangular_sum(f), a)
        }

        // And double_sum(f) = a
        double_sum(f) = double_limit(rectangular_sum(f))
        double_sum(f) = a

        // Now we show converges_to(square_sum(f), a)
        forall(eps: Real) {
            if eps.is_positive {
                // By double_converges_to, there exists N such that
                // for all i >= N and j >= N, rectangular_sum(f, i, j) is within eps of a
                let n: Nat satisfy {
                    double_limit_condition(rectangular_sum(f), a, n, eps)
                }

                // For all m >= n, square_sum(f, m) = rectangular_sum(f, m, m)
                forall(m: Nat) {
                    if n <= m {
                        // Since m >= n and m >= n, we have rectangular_sum(f, m, m).is_close(a, eps)
                        square_sum(f, m) = rectangular_sum(f, m, m)
                        rectangular_sum(f, m, m).is_close(a, eps)
                        square_sum(f, m).is_close(a, eps)
                    }
                }

                // This establishes tail_bound(square_sum(f), a, n, eps)
                tail_bound(square_sum(f), a, n, eps)
            }
        }

        converges_to(square_sum(f), a)
        converges_to(square_sum(f), double_sum(f))
    }
}

/// Alias endpoint: rectangular sums are doubly increasing for nonnegative terms.
theorem rectangular_sum_is_doubly_increasing_nonneg(f: (Nat, Nat) -> Real) {
    nonneg_fn_2(f) implies doubly_increasing(rectangular_sum(f))
} by {
    rectangular_sum_doubly_increasing(f)
}

/// Alias endpoint: row partial sums are monotone for nonnegative terms.
theorem row_partial_sum_is_monotone_nonneg(f: (Nat, Nat) -> Real, i: Nat) {
    nonneg_fn_2(f) implies is_monotone(partial(f(i)))
} by {
    row_partial_monotone(f, i)
}

/// Alias endpoint: column partial sums are monotone for nonnegative terms.
theorem col_partial_sum_is_monotone_nonneg(f: (Nat, Nat) -> Real, j: Nat) {
    nonneg_fn_2(f) implies is_monotone(partial(flip(f, j)))
} by {
    col_partial_monotone(f, j)
}

/// Alias endpoint: square partial sums are monotone for nonnegative terms.
theorem square_sum_seq_is_monotone_nonneg(f: (Nat, Nat) -> Real) {
    nonneg_fn_2(f) implies is_monotone(square_sum_seq(f))
} by {
    square_sum_is_monotone(f)
}


/// Alias endpoint: absolute-convergence Fubini row identity.
theorem fubini_rows_abs(f: (Nat, Nat) -> Real, s: Real) {
    double_image_is_supremum(rectangular_sum(abs_fn_2(f)), s)
    implies
    double_sum(f) = limit(partial(row_sums_seq(f)))
} by {
    fubini_rows(f, s)
}

/// Alias endpoint: absolute-convergence Fubini column identity.
theorem fubini_cols_abs(f: (Nat, Nat) -> Real, s: Real) {
    double_image_is_supremum(rectangular_sum(abs_fn_2(f)), s)
    implies
    double_sum(f) = limit(partial(col_sums_seq(f)))
} by {
    fubini_cols(f, s)
}
