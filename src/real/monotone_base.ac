/// Interval-relative monotonicity: definitions, unfolding lemmas, and the
/// elementary monotonicity facts used by the monotone function theorems and
/// the inverse function theorem.

from order import lte_refl, lte_trans, lt_trans, lt_imp_lte, not_lt_imp_gte, lte_antisymm,
    lt_of_lte_of_lt, lt_of_lt_of_lte, lt_imp_ne, lt_imp_ne_symm, not_lt_self, not_lte_imp_gt,
    lt_of_lte_of_ne, lt_of_lte_of_ne_symm
from order_set import closed_interval_set, closed_interval_set_contains_eq,
    closed_interval_set_lower_le, closed_interval_set_le_upper,
    closed_interval_set_contains_lower, closed_interval_set_contains_upper,
    closed_interval_set_subset_of_bounds
from order import closed_interval
from data.basic.set import Set, subset_contains
from data.basic.function_algebra import pointwise_neg
from algebra.add_ordered_group import neg_lt_neg, neg_le_neg, le_of_neg_le_neg
from real.continuity_base import Real, continuous
from real.intermediate_value import intermediate_value_closed_interval
from real.real_base import neg_neg, lt_add_pos
from real.continuity_pointwise import continuous_pointwise_neg

numerals Real

// =============================================================================
// Reverse intermediate value theorem
// =============================================================================

/// A continuous function on [lower, upper] takes every value between the
/// endpoint values, also when the values are in the decreasing order.
theorem intermediate_value_closed_interval_rev(
    f: Real -> Real, lower: Real, upper: Real, target: Real
) {
    continuous(f) and lower <= upper and f(upper) <= target and target <= f(lower)
    implies exists(point: Real) {
        closed_interval_set(lower, upper).contains(point) and f(point) = target
    }
} by {
    if continuous(f) and lower <= upper and f(upper) <= target and target <= f(lower) {
        continuous_pointwise_neg(f)
        continuous(pointwise_neg(f))
        neg_le_neg[Real](target, f(lower))
        -f(lower) <= -target
        neg_le_neg[Real](f(upper), target)
        -target <= -f(upper)
        pointwise_neg(f)(lower) = -f(lower)
        pointwise_neg(f)(upper) = -f(upper)
        pointwise_neg(f)(lower) <= -target
        -target <= pointwise_neg(f)(upper)
        intermediate_value_closed_interval(pointwise_neg(f), lower, upper, -target)
        let w: Real satisfy {
            closed_interval_set(lower, upper).contains(w) and pointwise_neg(f)(w) = -target
        }
        closed_interval_set(lower, upper).contains(w)
        pointwise_neg(f)(w) = -f(w)
        -f(w) = -target
        neg_neg(f(w))
        --f(w) = f(w)
        neg_neg(target)
        --target = target
        f(w) = target
        closed_interval_set(lower, upper).contains(w) and f(w) = target
        exists(point: Real) {
            closed_interval_set(lower, upper).contains(point) and f(point) = target
        }
    }
}
// =============================================================================
// Interval-relative monotonicity and injectivity
// =============================================================================

/// True if f is strictly increasing on the closed interval [lower, upper].
define increasing_on(f: Real -> Real, lower: Real, upper: Real) -> Bool {
    forall(x: Real, y: Real) {
        closed_interval_set(lower, upper).contains(x) and
        closed_interval_set(lower, upper).contains(y) and
        x < y implies f(x) < f(y)
    }
}

/// True if f is strictly decreasing on the closed interval [lower, upper].
define decreasing_on(f: Real -> Real, lower: Real, upper: Real) -> Bool {
    forall(x: Real, y: Real) {
        closed_interval_set(lower, upper).contains(x) and
        closed_interval_set(lower, upper).contains(y) and
        x < y implies f(y) < f(x)
    }
}

/// True if f is nondecreasing on the closed interval [lower, upper].
define nondecreasing_on(f: Real -> Real, lower: Real, upper: Real) -> Bool {
    forall(x: Real, y: Real) {
        closed_interval_set(lower, upper).contains(x) and
        closed_interval_set(lower, upper).contains(y) and
        x <= y implies f(x) <= f(y)
    }
}

/// True if f is nonincreasing on the closed interval [lower, upper].
define nonincreasing_on(f: Real -> Real, lower: Real, upper: Real) -> Bool {
    forall(x: Real, y: Real) {
        closed_interval_set(lower, upper).contains(x) and
        closed_interval_set(lower, upper).contains(y) and
        x <= y implies f(y) <= f(x)
    }
}

/// True if f is injective on the closed interval [lower, upper].
define injective_on(f: Real -> Real, lower: Real, upper: Real) -> Bool {
    forall(x: Real, y: Real) {
        closed_interval_set(lower, upper).contains(x) and
        closed_interval_set(lower, upper).contains(y) and
        f(x) = f(y) implies x = y
    }
}

/// Unfolding lemma for strict increase on a closed interval.
theorem increasing_on_iff(f: Real -> Real, lower: Real, upper: Real) {
    increasing_on(f, lower, upper) = forall(x: Real, y: Real) {
        closed_interval_set(lower, upper).contains(x) and
        closed_interval_set(lower, upper).contains(y) and
        x < y implies f(x) < f(y)
    }
} by {
    increasing_on(f, lower, upper) = forall(x: Real, y: Real) {
        closed_interval_set(lower, upper).contains(x) and
        closed_interval_set(lower, upper).contains(y) and
        x < y implies f(x) < f(y)
    }
}

/// Strict increase on [lower, upper] applies to an ordered pair of members.
theorem increasing_on_apply(f: Real -> Real, lower: Real, upper: Real, x: Real, y: Real) {
    increasing_on(f, lower, upper) and
    closed_interval_set(lower, upper).contains(x) and
    closed_interval_set(lower, upper).contains(y) and
    x < y
    implies f(x) < f(y)
} by {
    if increasing_on(f, lower, upper) and
       closed_interval_set(lower, upper).contains(x) and
       closed_interval_set(lower, upper).contains(y) and
       x < y {
        increasing_on_iff(f, lower, upper)
        closed_interval_set(lower, upper).contains(x) and
        closed_interval_set(lower, upper).contains(y) and
        x < y implies f(x) < f(y)
        f(x) < f(y)
    }
}

/// Unfolding lemma for strict decrease on a closed interval.
theorem decreasing_on_iff(f: Real -> Real, lower: Real, upper: Real) {
    decreasing_on(f, lower, upper) = forall(x: Real, y: Real) {
        closed_interval_set(lower, upper).contains(x) and
        closed_interval_set(lower, upper).contains(y) and
        x < y implies f(y) < f(x)
    }
} by {
    decreasing_on(f, lower, upper) = forall(x: Real, y: Real) {
        closed_interval_set(lower, upper).contains(x) and
        closed_interval_set(lower, upper).contains(y) and
        x < y implies f(y) < f(x)
    }
}

/// Strict decrease on [lower, upper] applies to an ordered pair of members.
theorem decreasing_on_apply(f: Real -> Real, lower: Real, upper: Real, x: Real, y: Real) {
    decreasing_on(f, lower, upper) and
    closed_interval_set(lower, upper).contains(x) and
    closed_interval_set(lower, upper).contains(y) and
    x < y
    implies f(y) < f(x)
} by {
    if decreasing_on(f, lower, upper) and
       closed_interval_set(lower, upper).contains(x) and
       closed_interval_set(lower, upper).contains(y) and
       x < y {
        decreasing_on_iff(f, lower, upper)
        closed_interval_set(lower, upper).contains(x) and
        closed_interval_set(lower, upper).contains(y) and
        x < y implies f(y) < f(x)
        f(y) < f(x)
    }
}

/// Unfolding lemma for nondecrease on a closed interval.
theorem nondecreasing_on_iff(f: Real -> Real, lower: Real, upper: Real) {
    nondecreasing_on(f, lower, upper) = forall(x: Real, y: Real) {
        closed_interval_set(lower, upper).contains(x) and
        closed_interval_set(lower, upper).contains(y) and
        x <= y implies f(x) <= f(y)
    }
} by {
    nondecreasing_on(f, lower, upper) = forall(x: Real, y: Real) {
        closed_interval_set(lower, upper).contains(x) and
        closed_interval_set(lower, upper).contains(y) and
        x <= y implies f(x) <= f(y)
    }
}

/// Nondecrease on [lower, upper] applies to an ordered pair of members.
theorem nondecreasing_on_apply(f: Real -> Real, lower: Real, upper: Real, x: Real, y: Real) {
    nondecreasing_on(f, lower, upper) and
    closed_interval_set(lower, upper).contains(x) and
    closed_interval_set(lower, upper).contains(y) and
    x <= y
    implies f(x) <= f(y)
} by {
    if nondecreasing_on(f, lower, upper) and
       closed_interval_set(lower, upper).contains(x) and
       closed_interval_set(lower, upper).contains(y) and
       x <= y {
        nondecreasing_on_iff(f, lower, upper)
        closed_interval_set(lower, upper).contains(x) and
        closed_interval_set(lower, upper).contains(y) and
        x <= y implies f(x) <= f(y)
        f(x) <= f(y)
    }
}

/// Unfolding lemma for nonincrease on a closed interval.
theorem nonincreasing_on_iff(f: Real -> Real, lower: Real, upper: Real) {
    nonincreasing_on(f, lower, upper) = forall(x: Real, y: Real) {
        closed_interval_set(lower, upper).contains(x) and
        closed_interval_set(lower, upper).contains(y) and
        x <= y implies f(y) <= f(x)
    }
} by {
    nonincreasing_on(f, lower, upper) = forall(x: Real, y: Real) {
        closed_interval_set(lower, upper).contains(x) and
        closed_interval_set(lower, upper).contains(y) and
        x <= y implies f(y) <= f(x)
    }
}

/// Nonincrease on [lower, upper] applies to an ordered pair of members.
theorem nonincreasing_on_apply(f: Real -> Real, lower: Real, upper: Real, x: Real, y: Real) {
    nonincreasing_on(f, lower, upper) and
    closed_interval_set(lower, upper).contains(x) and
    closed_interval_set(lower, upper).contains(y) and
    x <= y
    implies f(y) <= f(x)
} by {
    if nonincreasing_on(f, lower, upper) and
       closed_interval_set(lower, upper).contains(x) and
       closed_interval_set(lower, upper).contains(y) and
       x <= y {
        nonincreasing_on_iff(f, lower, upper)
        closed_interval_set(lower, upper).contains(x) and
        closed_interval_set(lower, upper).contains(y) and
        x <= y implies f(y) <= f(x)
        f(y) <= f(x)
    }
}

/// Unfolding lemma for injectivity on a closed interval.
theorem injective_on_iff(f: Real -> Real, lower: Real, upper: Real) {
    injective_on(f, lower, upper) = forall(x: Real, y: Real) {
        closed_interval_set(lower, upper).contains(x) and
        closed_interval_set(lower, upper).contains(y) and
        f(x) = f(y) implies x = y
    }
} by {
    injective_on(f, lower, upper) = forall(x: Real, y: Real) {
        closed_interval_set(lower, upper).contains(x) and
        closed_interval_set(lower, upper).contains(y) and
        f(x) = f(y) implies x = y
    }
}

/// Injectivity on [lower, upper] applies to two members with equal values.
theorem injective_on_apply(f: Real -> Real, lower: Real, upper: Real, x: Real, y: Real) {
    injective_on(f, lower, upper) and
    closed_interval_set(lower, upper).contains(x) and
    closed_interval_set(lower, upper).contains(y) and
    f(x) = f(y)
    implies x = y
} by {
    if injective_on(f, lower, upper) and
       closed_interval_set(lower, upper).contains(x) and
       closed_interval_set(lower, upper).contains(y) and
       f(x) = f(y) {
        injective_on_iff(f, lower, upper)
        closed_interval_set(lower, upper).contains(x) and
        closed_interval_set(lower, upper).contains(y) and
        f(x) = f(y) implies x = y
        x = y
    }
}

/// A strictly increasing function sends equal values back to equal arguments.
theorem increasing_on_values_imp_args_eq(f: Real -> Real, a: Real, b: Real, x: Real, y: Real) {
    increasing_on(f, a, b) and
    closed_interval_set(a, b).contains(x) and closed_interval_set(a, b).contains(y) and
    f(x) = f(y)
    implies x = y
} by {
    if increasing_on(f, a, b) and
       closed_interval_set(a, b).contains(x) and closed_interval_set(a, b).contains(y) and
       f(x) = f(y) {
        if x < y {
            increasing_on_apply(f, a, b, x, y)
            f(x) < f(y)
            f(x) = f(y)
            false
        }
        if y < x {
            increasing_on_apply(f, a, b, y, x)
            f(y) < f(x)
            f(x) = f(y)
            false
        }
        not_lt_imp_gte[Real](x, y)
        y <= x
        not_lt_imp_gte[Real](y, x)
        x <= y
        lte_antisymm[Real](x, y)
        x = y
    }
}
/// A point of an inner closed interval belongs to the ambient closed interval.
theorem closed_interval_inner_member(
    a: Real, b: Real, u: Real, v: Real, w: Real
) {
    a <= u and v <= b and closed_interval_set(u, v).contains(w)
    implies closed_interval_set(a, b).contains(w)
} by {
    if a <= u and v <= b and closed_interval_set(u, v).contains(w) {
        closed_interval_set_subset_of_bounds(a, u, v, b)
        closed_interval_set(u, v).subset(closed_interval_set(a, b))
        subset_contains(closed_interval_set(u, v), closed_interval_set(a, b), w)
        closed_interval_set(u, v).contains(w)
        closed_interval_set(a, b).contains(w)
    }
}
/// Negating a function preserves injectivity on a closed interval.
theorem injective_on_pointwise_neg(f: Real -> Real, a: Real, b: Real) {
    injective_on(f, a, b) implies injective_on(pointwise_neg(f), a, b)
} by {
    if injective_on(f, a, b) {
        forall(x: Real, y: Real) {
            if closed_interval_set(a, b).contains(x) and
               closed_interval_set(a, b).contains(y) and
               pointwise_neg(f)(x) = pointwise_neg(f)(y) {
                pointwise_neg(f)(x) = -f(x)
                pointwise_neg(f)(y) = -f(y)
                -f(x) = -f(y)
                neg_neg(f(x))
                --f(x) = f(x)
                neg_neg(f(y))
                --f(y) = f(y)
                f(x) = f(y)
                injective_on_apply(f, a, b, x, y)
                x = y
            }
        }
        injective_on(pointwise_neg(f), a, b)
    }
}
/// A strictly increasing function sends ordered values back to ordered arguments.
theorem increasing_on_values_imp_args_lt(f: Real -> Real, a: Real, b: Real, x: Real, y: Real) {
    increasing_on(f, a, b) and
    closed_interval_set(a, b).contains(x) and closed_interval_set(a, b).contains(y) and
    f(x) < f(y)
    implies x < y
} by {
    if increasing_on(f, a, b) and
       closed_interval_set(a, b).contains(x) and closed_interval_set(a, b).contains(y) and
       f(x) < f(y) {
        if not x < y {
            not_lt_imp_gte[Real](x, y)
            y <= x
            if y < x {
                increasing_on_apply(f, a, b, y, x)
                f(y) < f(x)
                f(x) < f(y)
                false
            } else {
                not_lt_imp_gte[Real](y, x)
                x <= y
                lte_antisymm[Real](x, y)
                x = y
                f(x) = f(y)
                f(x) < f(y)
                false
            }
        }
        x < y
    }
}
/// A point between a and b belongs to the closed interval set with those endpoints.
theorem closed_interval_set_contains_of_bounds(a: Real, b: Real, x: Real) {
    a <= x and x <= b implies closed_interval_set(a, b).contains(x)
} by {
    if a <= x and x <= b {
        closed_interval(a, b, x)
        closed_interval_set_contains_eq(a, b, x)
        closed_interval_set(a, b).contains(x)
    }
}
