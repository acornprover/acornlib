from nat import Nat, lte_ref, lte_trans
from order import max_imp_gte
from data.basic.set import Set, intersection_contains_intro, subset_contains, union_contains_left,
    union_contains_right
from real.real_field import Real

/// True if every term of a real sequence lies in a set.
define seq_in_real_set(s: Set[Real], a: Nat -> Real) -> Bool {
    forall(n: Nat) {
        s.contains(a(n))
    }
}

/// True if all sufficiently late terms of a real sequence lie in a set.
define seq_eventually_in_real_set(s: Set[Real], a: Nat -> Real) -> Bool {
    exists(n0: Nat) {
        forall(n: Nat) {
            n0 <= n implies s.contains(a(n))
        }
    }
}

/// True if arbitrarily late terms of a real sequence lie in a set.
define seq_frequently_in_real_set(s: Set[Real], a: Nat -> Real) -> Bool {
    forall(n_start: Nat) {
        exists(n_witness: Nat) {
            n_start <= n_witness and s.contains(a(n_witness))
        }
    }
}

/// Frequent membership gives a witness after any starting index.
theorem seq_frequently_in_real_set_at(s: Set[Real], a: Nat -> Real, n_start: Nat) {
    seq_frequently_in_real_set(s, a) implies exists(n_witness: Nat) {
        n_start <= n_witness and s.contains(a(n_witness))
    }
} by {
    if seq_frequently_in_real_set(s, a) {
        seq_frequently_in_real_set(s, a) = forall(k: Nat) {
            exists(n_witness: Nat) {
                k <= n_witness and s.contains(a(n_witness))
            }
        }
        exists(n_witness: Nat) {
            n_start <= n_witness and s.contains(a(n_witness))
        }
    }
}

/// A sequence contained in a real set has each term in that set.
theorem seq_in_real_set_at(s: Set[Real], a: Nat -> Real, n: Nat) {
    seq_in_real_set(s, a) implies s.contains(a(n))
} by {
    if seq_in_real_set(s, a) {
        seq_in_real_set(s, a) = forall(k: Nat) {
            s.contains(a(k))
        }
        s.contains(a(n))
    }
}

/// Pointwise membership makes a sequence contained in a real set.
theorem seq_in_real_set_intro(s: Set[Real], a: Nat -> Real) {
    forall(n: Nat) { s.contains(a(n)) } implies seq_in_real_set(s, a)
}

/// Sequence membership is preserved by set inclusion.
theorem seq_in_real_set_of_subset(s: Set[Real], t: Set[Real], a: Nat -> Real) {
    s.subset(t) and seq_in_real_set(s, a) implies seq_in_real_set(t, a)
} by {
    if s.subset(t) and seq_in_real_set(s, a) {
        forall(n: Nat) {
            seq_in_real_set_at(s, a, n)
            subset_contains(s, t, a(n))
            t.contains(a(n))
        }
        seq_in_real_set(t, a)
    }
}

/// If every term lies in two sets, every term lies in their intersection.
theorem seq_in_real_set_intersection(s: Set[Real], t: Set[Real], a: Nat -> Real) {
    seq_in_real_set(s, a) and seq_in_real_set(t, a) implies seq_in_real_set(s.intersection(t), a)
} by {
    if seq_in_real_set(s, a) and seq_in_real_set(t, a) {
        forall(n: Nat) {
            seq_in_real_set_at(s, a, n)
            seq_in_real_set_at(t, a, n)
            s.contains(a(n))
            t.contains(a(n))
            intersection_contains_intro(s, t, a(n))
            s.intersection(t).contains(a(n))
        }
        seq_in_real_set(s.intersection(t), a)
    }
}

/// If every term lies in the left set, every term lies in the union.
theorem seq_in_real_set_union_left(s: Set[Real], t: Set[Real], a: Nat -> Real) {
    seq_in_real_set(s, a) implies seq_in_real_set(s.union(t), a)
} by {
    if seq_in_real_set(s, a) {
        forall(n: Nat) {
            seq_in_real_set_at(s, a, n)
            union_contains_left(s, t, a(n))
            s.union(t).contains(a(n))
        }
        seq_in_real_set(s.union(t), a)
    }
}

/// If every term lies in the right set, every term lies in the union.
theorem seq_in_real_set_union_right(s: Set[Real], t: Set[Real], a: Nat -> Real) {
    seq_in_real_set(t, a) implies seq_in_real_set(s.union(t), a)
} by {
    if seq_in_real_set(t, a) {
        forall(n: Nat) {
            seq_in_real_set_at(t, a, n)
            union_contains_right(s, t, a(n))
            s.union(t).contains(a(n))
        }
        seq_in_real_set(s.union(t), a)
    }
}

/// A sequence contained in a real set is eventually in that set.
theorem seq_in_real_set_eventually(s: Set[Real], a: Nat -> Real) {
    seq_in_real_set(s, a) implies seq_eventually_in_real_set(s, a)
} by {
    if seq_in_real_set(s, a) {
        forall(n: Nat) {
            if Nat.0 <= n {
                seq_in_real_set_at(s, a, n)
            }
        }
        exists(n0: Nat) {
            forall(n: Nat) {
                n0 <= n implies s.contains(a(n))
            }
        }
        seq_eventually_in_real_set(s, a)
    }
}

/// A sequence contained in a real set is frequently in that set.
theorem seq_in_real_set_frequently(s: Set[Real], a: Nat -> Real) {
    seq_in_real_set(s, a) implies seq_frequently_in_real_set(s, a)
} by {
    if seq_in_real_set(s, a) {
        forall(n_start: Nat) {
            seq_in_real_set_at(s, a, n_start)
            exists(n_witness: Nat) {
                n_start <= n_witness and s.contains(a(n_witness))
            }
        }
        seq_frequently_in_real_set(s, a)
    }
}

/// Eventual membership is preserved by set inclusion.
theorem seq_eventually_in_real_set_of_subset(s: Set[Real], t: Set[Real], a: Nat -> Real) {
    s.subset(t) and seq_eventually_in_real_set(s, a) implies seq_eventually_in_real_set(t, a)
} by {
    if s.subset(t) and seq_eventually_in_real_set(s, a) {
        let n0: Nat satisfy {
            forall(n: Nat) {
                n0 <= n implies s.contains(a(n))
            }
        }
        forall(n: Nat) {
            if n0 <= n {
                s.contains(a(n))
                subset_contains(s, t, a(n))
                t.contains(a(n))
            }
        }
        exists(m: Nat) {
            forall(n: Nat) {
                m <= n implies t.contains(a(n))
            }
        }
        seq_eventually_in_real_set(t, a)
    }
}

/// Eventual membership in two sets gives eventual membership in their intersection.
theorem seq_eventually_in_real_set_intersection(s: Set[Real], t: Set[Real], a: Nat -> Real) {
    seq_eventually_in_real_set(s, a) and seq_eventually_in_real_set(t, a)
    implies seq_eventually_in_real_set(s.intersection(t), a)
} by {
    if seq_eventually_in_real_set(s, a) and seq_eventually_in_real_set(t, a) {
        let ns: Nat satisfy {
            forall(n: Nat) {
                ns <= n implies s.contains(a(n))
            }
        }
        let nt: Nat satisfy {
            forall(n: Nat) {
                nt <= n implies t.contains(a(n))
            }
        }
        let n0 = ns.max(nt)
        max_imp_gte[Nat](ns, nt)
        n0 >= ns and n0 >= nt
        ns <= n0
        nt <= n0
        forall(n: Nat) {
            if n0 <= n {
                ns <= n
                nt <= n
                s.contains(a(n))
                t.contains(a(n))
                intersection_contains_intro(s, t, a(n))
                s.intersection(t).contains(a(n))
            }
        }
        exists(m: Nat) {
            forall(n: Nat) {
                m <= n implies s.intersection(t).contains(a(n))
            }
        }
        seq_eventually_in_real_set(s.intersection(t), a)
    }
}

/// Eventual membership in the left set gives eventual membership in the union.
theorem seq_eventually_in_real_set_union_left(s: Set[Real], t: Set[Real], a: Nat -> Real) {
    seq_eventually_in_real_set(s, a) implies seq_eventually_in_real_set(s.union(t), a)
} by {
    if seq_eventually_in_real_set(s, a) {
        let n0: Nat satisfy {
            forall(n: Nat) {
                n0 <= n implies s.contains(a(n))
            }
        }
        forall(n: Nat) {
            if n0 <= n {
                s.contains(a(n))
                union_contains_left(s, t, a(n))
                s.union(t).contains(a(n))
            }
        }
        exists(m: Nat) {
            forall(n: Nat) {
                m <= n implies s.union(t).contains(a(n))
            }
        }
        seq_eventually_in_real_set(s.union(t), a)
    }
}

/// Eventual membership in the right set gives eventual membership in the union.
theorem seq_eventually_in_real_set_union_right(s: Set[Real], t: Set[Real], a: Nat -> Real) {
    seq_eventually_in_real_set(t, a) implies seq_eventually_in_real_set(s.union(t), a)
} by {
    if seq_eventually_in_real_set(t, a) {
        let n0: Nat satisfy {
            forall(n: Nat) {
                n0 <= n implies t.contains(a(n))
            }
        }
        forall(n: Nat) {
            if n0 <= n {
                t.contains(a(n))
                union_contains_right(s, t, a(n))
                s.union(t).contains(a(n))
            }
        }
        exists(m: Nat) {
            forall(n: Nat) {
                m <= n implies s.union(t).contains(a(n))
            }
        }
        seq_eventually_in_real_set(s.union(t), a)
    }
}

/// Eventual membership gives frequent membership.
theorem seq_eventually_in_real_set_frequently(s: Set[Real], a: Nat -> Real) {
    seq_eventually_in_real_set(s, a) implies seq_frequently_in_real_set(s, a)
} by {
    if seq_eventually_in_real_set(s, a) {
        let n0: Nat satisfy {
            forall(n: Nat) {
                n0 <= n implies s.contains(a(n))
            }
        }
        forall(n_start: Nat) {
            let n_witness = n_start.max(n0)
            max_imp_gte[Nat](n_start, n0)
            n_witness >= n_start and n_witness >= n0
            n_start <= n_witness
            n0 <= n_witness
            s.contains(a(n_witness))
            exists(n: Nat) {
                n_start <= n and s.contains(a(n))
            }
        }
        seq_frequently_in_real_set(s, a)
    }
}

/// Frequent membership is preserved by set inclusion.
theorem seq_frequently_in_real_set_of_subset(s: Set[Real], t: Set[Real], a: Nat -> Real) {
    s.subset(t) and seq_frequently_in_real_set(s, a) implies seq_frequently_in_real_set(t, a)
} by {
    if s.subset(t) and seq_frequently_in_real_set(s, a) {
        forall(n_start: Nat) {
            let n_witness: Nat satisfy {
                n_start <= n_witness and s.contains(a(n_witness))
            }
            subset_contains(s, t, a(n_witness))
            t.contains(a(n_witness))
            exists(n: Nat) {
                n_start <= n and t.contains(a(n))
            }
        }
        seq_frequently_in_real_set(t, a)
    }
}

/// Frequent membership in one set and eventual membership in another give frequent membership in their intersection.
theorem seq_frequently_eventually_in_real_set_intersection(s: Set[Real], t: Set[Real], a: Nat -> Real) {
    seq_frequently_in_real_set(s, a) and seq_eventually_in_real_set(t, a)
    implies seq_frequently_in_real_set(s.intersection(t), a)
} by {
    if seq_frequently_in_real_set(s, a) and seq_eventually_in_real_set(t, a) {
        let nt: Nat satisfy {
            forall(n: Nat) {
                nt <= n implies t.contains(a(n))
            }
        }
        forall(n_start: Nat) {
            let m = n_start.max(nt)
            max_imp_gte[Nat](n_start, nt)
            m >= n_start and m >= nt
            n_start <= m
            nt <= m
            let n_witness: Nat satisfy {
                m <= n_witness and s.contains(a(n_witness))
            }
            n_start <= n_witness
            nt <= n_witness
            t.contains(a(n_witness))
            intersection_contains_intro(s, t, a(n_witness))
            s.intersection(t).contains(a(n_witness))
            exists(n: Nat) {
                n_start <= n and s.intersection(t).contains(a(n))
            }
        }
        seq_frequently_in_real_set(s.intersection(t), a)
    }
}
