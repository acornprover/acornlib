/// Double limits for functions from (Nat, Nat) to Real.
from nat import Nat, lte_trans
from real.real_base import Real
from real.real_series import is_increasing, is_upper_bound, monotone_convergence_principle, seq_lte_preserves_limit, increasing_convergent_bounded_by_limit
from real.real_series import increasing_is_monotone, monotone_bounded_above_converges
from real.real_series import seq_lte
from real.real_series import distant_increasing
from real.real_seq import converges, converges_to, limit, ub_imp_limit_lte, eventual_ub, converges_imp_converges_to, converges_to_unique
from real.rectangular_sum import add_fn_2, sub_fn_2
from data.basic.functions import flip
from order import is_monotone

/// The condition that f is within eps of val for all indices beyond n.
define double_limit_condition(f: (Nat, Nat) -> Real, val: Real, n: Nat, eps: Real) -> Bool {
    forall(i: Nat, j: Nat) {
        n <= i and n <= j implies f(i, j).is_close(val, eps)
    }
}

/// f converges to val.
define double_converges_to(f: (Nat, Nat) -> Real, val: Real) -> Bool {
    forall(eps: Real) {
        eps.is_positive implies exists(n: Nat) {
            double_limit_condition(f, val, n, eps)
        }
    }
}

/// f converges to some value.
define double_converges(f: (Nat, Nat) -> Real) -> Bool {
    exists(val: Real) {
        double_converges_to(f, val)
    }
}

/// The limit of f. Only meaningful when f converges.
let double_limit(f: (Nat, Nat) -> Real) -> val: Real satisfy {
    double_converges(f) implies double_converges_to(f, val)
}

/// A function is doubly increasing when it is increasing in both arguments.
/// That is, f(i, j) <= f(i, j+1) and f(i, j) <= f(i+1, j) for all i, j.
define doubly_increasing(f: (Nat, Nat) -> Real) -> Bool {
    forall(i: Nat, j: Nat) {
        f(i, j) <= f(i, j.suc) and f(i, j) <= f(i.suc, j)
    }
}

/// If f is doubly increasing, then for any fixed first argument i,
/// the function j -> f(i, j) is increasing.
theorem doubly_increasing_right(f: (Nat, Nat) -> Real, i: Nat) {
    doubly_increasing(f) implies is_increasing(function(j: Nat) { f(i, j) })
} by {
    if doubly_increasing(f) {
        forall(j: Nat) {
            f(i, j) <= f(i, j.suc)
        }
    }
}

/// If f is doubly increasing, then each row is monotone.
theorem doubly_increasing_right_monotone(f: (Nat, Nat) -> Real, i: Nat) {
    doubly_increasing(f) implies is_monotone(function(j: Nat) { f(i, j) })
} by {
    if doubly_increasing(f) {
        doubly_increasing_right(f, i)
        is_increasing(function(j: Nat) { f(i, j) })
        increasing_is_monotone(function(j: Nat) { f(i, j) })
        is_monotone(function(j: Nat) { f(i, j) })
    }
}

/// If f is doubly increasing, then for any fixed second argument j,
/// the function i -> f(i, j) is increasing.
theorem doubly_increasing_left(f: (Nat, Nat) -> Real, j: Nat) {
    doubly_increasing(f) implies is_increasing(function(i: Nat) { f(i, j) })
} by {
    if doubly_increasing(f) {
        forall(i: Nat) {
            f(i, j) <= f(i.suc, j)
        }
    }
}

/// If f is doubly increasing, then each column is monotone.
theorem doubly_increasing_left_monotone(f: (Nat, Nat) -> Real, j: Nat) {
    doubly_increasing(f) implies is_monotone(function(i: Nat) { f(i, j) })
} by {
    if doubly_increasing(f) {
        doubly_increasing_left(f, j)
        is_increasing(function(i: Nat) { f(i, j) })
        increasing_is_monotone(function(i: Nat) { f(i, j) })
        is_monotone(function(i: Nat) { f(i, j) })
    }
}

/// If f is doubly increasing, then flip(f, j) is increasing.
/// This is equivalent to doubly_increasing_left but using the flip function.
theorem doubly_increasing_flip(f: (Nat, Nat) -> Real, j: Nat) {
    doubly_increasing(f) implies is_increasing(function(i: Nat) { flip(f, j, i) })
} by {
    if doubly_increasing(f) {
        forall(i: Nat) {
            flip(f, j, i) <= flip(f, j, i.suc)
        }
    }
}

/// If f is doubly increasing, then the flipped column is monotone.
theorem doubly_increasing_flip_monotone(f: (Nat, Nat) -> Real, j: Nat) {
    doubly_increasing(f) implies is_monotone(function(i: Nat) { flip(f, j, i) })
} by {
    if doubly_increasing(f) {
        doubly_increasing_flip(f, j)
        is_increasing(function(i: Nat) { flip(f, j, i) })
        increasing_is_monotone(function(i: Nat) { flip(f, j, i) })
        is_monotone(function(i: Nat) { flip(f, j, i) })
    }
}

/// If f is doubly increasing and i1 <= i2, then f(i1, j) <= f(i2, j).
theorem doubly_increasing_first_mono(f: (Nat, Nat) -> Real, i1: Nat, i2: Nat, j: Nat) {
    doubly_increasing(f) and i1 <= i2 implies f(i1, j) <= f(i2, j)
} by {
    if doubly_increasing(f) and i1 <= i2 {
        // Use the fact that function(i: Nat) { f(i, j) } is increasing
        // and apply distant_increasing
        let g = function(i: Nat) { f(i, j) }
        forall(i: Nat) {
            f(i, j) <= f(i.suc, j)
            g(i) <= g(i.suc)
        }
        is_increasing(g)
        g(i1) <= g(i2)
        f(i1, j) <= f(i2, j)
    }
}

/// If f is doubly increasing and j1 <= j2, then f(i, j1) <= f(i, j2).
theorem doubly_increasing_second_mono(f: (Nat, Nat) -> Real, i: Nat, j1: Nat, j2: Nat) {
    doubly_increasing(f) and j1 <= j2 implies f(i, j1) <= f(i, j2)
} by {
    if doubly_increasing(f) and j1 <= j2 {
        // Use the fact that function(j: Nat) { f(i, j) } is increasing
        // and apply distant_increasing
        let g = function(j: Nat) { f(i, j) }
        forall(j: Nat) {
            f(i, j) <= f(i, j.suc)
            g(j) <= g(j.suc)
        }
        is_increasing(g)
        is_increasing(g) and j1 <= j2
        g(j1) <= g(j2)
        f(i, j1) <= f(i, j2)
        is_increasing(f(i))
        is_increasing(f(i)) and j1 <= j2
        f(i)(j1) <= f(i)(j2)
        f(i, j1) <= f(i, j2)
        let h = f(i)
        is_increasing(h)
        is_increasing(h) and j1 <= j2
        h(j1) <= h(j2)
        h(j1) = f(i, j1)
        h(j2) = f(i, j2)
        f(i, j1) <= f(i, j2)
        is_increasing(function(j: Nat) { f(i, j) })
        f(i, j1) <= f(i, j2)
    }
    f(i, j1) <= f(i, j2)
}

/// If f is doubly increasing, i1 <= i2, and j1 <= j2, then f(i1, j1) <= f(i2, j2).
theorem doubly_increasing_monotone(f: (Nat, Nat) -> Real, i1: Nat, i2: Nat, j1: Nat, j2: Nat) {
    doubly_increasing(f) and i1 <= i2 and j1 <= j2 implies f(i1, j1) <= f(i2, j2)
} by {
    if doubly_increasing(f) and i1 <= i2 and j1 <= j2 {
        // f(i1, j1) <= f(i2, j1) by doubly_increasing_first_mono
        f(i1, j1) <= f(i2, j1)
        // f(i2, j1) <= f(i2, j2) by doubly_increasing_second_mono
        f(i2, j1) <= f(i2, j2)
        // By transitivity
        f(i1, j1) <= f(i2, j2)
    }
}

/// True if x is in the image of f.
define double_image(f: (Nat, Nat) -> Real, x: Real) -> Bool {
    exists(i: Nat, j: Nat) {
        f(i, j) = x
    }
}

/// True if b is an upper bound for all values of f.
define double_is_upper_bound(f: (Nat, Nat) -> Real, b: Real) -> Bool {
    forall(i: Nat, j: Nat) {
        f(i, j) <= b
    }
}

/// A pointwise upper bound is a double upper bound.
theorem double_is_upper_bound_from_forall(f: (Nat, Nat) -> Real, b: Real) {
    (forall(i: Nat, j: Nat) {
        f(i, j) <= b
    })
    implies
    double_is_upper_bound(f, b)
}

/// True if s is the supremum of the image of f.
define double_image_is_supremum(f: (Nat, Nat) -> Real, s: Real) -> Bool {
    double_is_upper_bound(f, s) and
    forall(b: Real) {
        b < s implies not double_is_upper_bound(f, b)
    }
}

/// The defining parts determine the double-image supremum.
theorem double_image_is_supremum_from_parts(f: (Nat, Nat) -> Real, s: Real) {
    double_is_upper_bound(f, s) and
    forall(b: Real) {
        b < s implies not double_is_upper_bound(f, b)
    }
    implies
    double_image_is_supremum(f, s)
}

/// If f is doubly increasing and bounded above, then f converges to the
/// supremum of its image.
theorem doubly_increasing_bounded_converges(f: (Nat, Nat) -> Real, s: Real) {
    doubly_increasing(f) and double_image_is_supremum(f, s)
    implies
    double_converges_to(f, s)
} by {
    if doubly_increasing(f) and double_image_is_supremum(f, s) {
        // Need to show: for every eps > 0, there exist n such that
        // for all i >= n and j >= n, f(i, j) is within eps of s

        forall(eps: Real) {
            if eps.is_positive {
                // By the supremum property, s is an upper bound
                double_is_upper_bound(f, s)

                // Also, s - eps is not an upper bound (since s is the least upper bound)
                // So there exist a0, b0 with f(a0, b0) > s - eps
                s - eps < s
                not double_is_upper_bound(f, s - eps)

                // Therefore, there exist a0, b0 such that f(a0, b0) > s - eps
                exists(k0: Nat, k1: Nat) {
                    not f(k0, k1) <= s - eps
                }
                exists(k0: Nat, k1: Nat) {
                    f(k0, k1) > s - eps
                }
                let (a0: Nat, b0: Nat) satisfy {
                    f(a0, b0) > s - eps
                }

                // Now, for any i >= a0 and j >= b0, by doubly increasing property:
                forall(i: Nat, j: Nat) {
                    if a0 <= i and b0 <= j {
                        // f(a0, b0) <= f(i, j) by monotonicity
                        f(a0, b0) <= f(i, j)

                        // So s - eps < f(a0, b0) <= f(i, j)
                        s - eps < f(i, j)

                        // And f(i, j) <= s since s is an upper bound
                        f(i, j) <= s

                        // Therefore s - eps < f(i, j) <= s
                        // which means |f(i, j) - s| < eps
                        f(i, j).is_close(s, eps)
                    }
                }

                // Take n = max(a0, b0) to ensure n <= i and n <= j
                let n = a0.max(b0)

                forall(i: Nat, j: Nat) {
                    if n <= i and n <= j {
                        // n = max(a0, b0), so a0 <= n and b0 <= n
                        // Therefore a0 <= n <= i and b0 <= n <= j
                        a0 <= n
                        b0 <= n
                        a0 <= i
                        b0 <= j

                        f(i, j).is_close(s, eps)
                    }
                }

                exists(k0: Nat, k1: Nat) {
                    n <= k0 and n <= k1 and not f(k0, k1).is_close(s, eps)
                } or double_limit_condition(f, s, n, eps)
                double_limit_condition(f, s, n, eps)
            }
        }

        double_converges_to(f, s)
    }
}

/// For a fixed i, the row sequence j -> f(i, j) converges.
define row_converges(f: (Nat, Nat) -> Real, i: Nat) -> Bool {
    converges(f(i))
}

/// The limit of the row sequence for fixed i.
define row_limit(f: (Nat, Nat) -> Real, i: Nat) -> Real {
    limit(f(i))
}

/// For a fixed j, the column sequence i -> f(i, j) converges.
define col_converges(f: (Nat, Nat) -> Real, j: Nat) -> Bool {
    converges(flip(f, j))
}

/// The limit of the column sequence for fixed j.
define col_limit(f: (Nat, Nat) -> Real, j: Nat) -> Real {
    limit(flip(f, j))
}

/// All row limits exist.
define all_rows_converge(f: (Nat, Nat) -> Real) -> Bool {
    forall(i: Nat) {
        row_converges(f, i)
    }
}

/// All column limits exist.
define all_cols_converge(f: (Nat, Nat) -> Real) -> Bool {
    forall(j: Nat) {
        col_converges(f, j)
    }
}

/// The iterated limit lim_{i→∞} lim_{j→∞} f(i, j).
/// This is the limit as i→∞ of the row limits.
define iterated_limit_rows(f: (Nat, Nat) -> Real) -> Real {
    limit(row_limit(f))
}

/// The iterated limit lim_{j→∞} lim_{i→∞} f(i, j).
/// This is the limit as j→∞ of the column limits.
define iterated_limit_cols(f: (Nat, Nat) -> Real) -> Real {
    limit(col_limit(f))
}

/// row_limit unfolds to the underlying limit.
theorem row_limit_unfold(f: (Nat, Nat) -> Real, i: Nat) {
    row_limit(f, i) = limit(f(i))
}

/// col_limit unfolds to the underlying limit.
theorem col_limit_unfold(f: (Nat, Nat) -> Real, j: Nat) {
    col_limit(f, j) = limit(flip(f, j))
}

/// row_converges unfolds to the underlying convergence.
theorem row_converges_unfold(f: (Nat, Nat) -> Real, i: Nat) {
    row_converges(f, i) = converges(f(i))
}

/// col_converges unfolds to the underlying convergence.
theorem col_converges_unfold(f: (Nat, Nat) -> Real, j: Nat) {
    col_converges(f, j) = converges(flip(f, j))
}

/// For a doubly increasing bounded function, each row sequence converges.
theorem row_sequence_converges(f: (Nat, Nat) -> Real, i: Nat, b: Real) {
    doubly_increasing(f) and double_is_upper_bound(f, b)
    implies
    converges(f(i))
} by {
    if doubly_increasing(f) and double_is_upper_bound(f, b) {
        // f(i) is increasing and bounded, so it converges by monotone convergence
        is_increasing(f(i))

        // f(i, j) <= b for all j
        forall(j: Nat) {
            f(i, j) <= b
            f(i)(j) <= b
        }
        forall(j: Nat) {
            if Nat.zero <= j {
                f(i)(j) <= b
            }
        }
        exists(n: Nat) {
            forall(j: Nat) {
                n <= j implies f(i)(j) <= b
            }
        }
        is_upper_bound(f(i), b)

        // Apply monotone convergence principle
        converges(f(i))
    }
}

/// For a doubly increasing bounded function, each column sequence converges.
theorem col_sequence_converges(f: (Nat, Nat) -> Real, j: Nat, b: Real) {
    doubly_increasing(f) and double_is_upper_bound(f, b)
    implies
    converges(flip(f, j))
} by {
    if doubly_increasing(f) and double_is_upper_bound(f, b) {
        // flip(f, j) is increasing and bounded, so it converges by monotone convergence
        is_increasing(flip(f, j))

        // flip(f, j, i) = f(i, j) <= b for all i
        forall(i: Nat) {
            flip(f, j, i) <= b
            flip(f, j)(i) <= b
        }
        forall(i: Nat) {
            if Nat.zero <= i {
                flip(f, j)(i) <= b
            }
        }
        exists(n: Nat) {
            forall(i: Nat) {
                n <= i implies flip(f, j)(i) <= b
            }
        }
        is_upper_bound(flip(f, j), b)

        // Apply monotone convergence principle
        converges(flip(f, j))
    }
}

/// The sequence of row limits is increasing.
theorem row_limits_increasing(f: (Nat, Nat) -> Real, b: Real, i: Nat) {
    doubly_increasing(f) and double_is_upper_bound(f, b)
    implies
    limit(f(i)) <= limit(f(i.suc))
} by {
    if doubly_increasing(f) and double_is_upper_bound(f, b) {
        // Both sequences converge
        converges(f(i))
        converges(f(i.suc))

        // f(i, j) <= f(i.suc, j) for all j
        forall(j: Nat) {
            f(i, j) <= f(i.suc, j)
        }
        seq_lte(f(i), f(i.suc))

        // By seq_lte_preserves_limit
        limit(f(i)) <= limit(f(i.suc))
    }
}

/// The sequence of column limits is increasing.
theorem col_limits_increasing(f: (Nat, Nat) -> Real, b: Real, j: Nat) {
    doubly_increasing(f) and double_is_upper_bound(f, b)
    implies
    limit(flip(f, j)) <= limit(flip(f, j.suc))
} by {
    if doubly_increasing(f) and double_is_upper_bound(f, b) {
        // Both sequences converge
        converges(flip(f, j))
        converges(flip(f, j.suc))

        // flip(f, j, i) = f(i, j) <= f(i, j.suc) = flip(f, j.suc, i) for all i
        forall(i: Nat) {
            f(i, j) <= f(i, j.suc)
            flip(f, j, i) <= flip(f, j.suc, i)
        }
        seq_lte(flip(f, j), flip(f, j.suc))

        // By seq_lte_preserves_limit
        limit(flip(f, j)) <= limit(flip(f, j.suc))
    }
}

/// Each row limit is bounded by b.
theorem row_limit_bounded(f: (Nat, Nat) -> Real, i: Nat, b: Real) {
    doubly_increasing(f) and double_is_upper_bound(f, b)
    implies
    limit(f(i)) <= b
} by {
    if doubly_increasing(f) and double_is_upper_bound(f, b) {
        // f(i) is increasing
        is_increasing(f(i))

        // f(i, j) <= b for all j
        forall(j: Nat) {
            f(i, j) <= b
            f(i)(j) <= b
        }
        forall(j: Nat) {
            if Nat.zero <= j {
                f(i)(j) <= b
            }
        }
        exists(n: Nat) {
            forall(j: Nat) {
                n <= j implies f(i)(j) <= b
            }
        }
        is_upper_bound(f(i), b)

        // f(i) converges
        converges(f(i))

        // By increasing_convergent_bounded_by_limit
        eventual_ub(f(i), b)
        limit(f(i)) <= b
    }
}

/// Each column limit is bounded by b.
theorem col_limit_bounded(f: (Nat, Nat) -> Real, j: Nat, b: Real) {
    doubly_increasing(f) and double_is_upper_bound(f, b)
    implies
    limit(flip(f, j)) <= b
} by {
    if doubly_increasing(f) and double_is_upper_bound(f, b) {
        // flip(f, j) is increasing
        is_increasing(flip(f, j))

        // flip(f, j, i) = f(i, j) <= b for all i
        forall(i: Nat) {
            flip(f, j, i) <= b
            flip(f, j)(i) <= b
        }
        forall(i: Nat) {
            if Nat.zero <= i {
                flip(f, j)(i) <= b
            }
        }
        exists(n: Nat) {
            forall(i: Nat) {
                n <= i implies flip(f, j)(i) <= b
            }
        }
        is_upper_bound(flip(f, j), b)

        // flip(f, j) converges
        converges(flip(f, j))

        // By ub_imp_limit_lte
        eventual_ub(flip(f, j), b)
        limit(flip(f, j)) <= b
    }
}

/// The sequence of row limits is increasing (full version).
theorem row_limits_is_increasing(f: (Nat, Nat) -> Real, b: Real) {
    doubly_increasing(f) and double_is_upper_bound(f, b)
    implies
    is_increasing(row_limit(f))
} by {
    if doubly_increasing(f) and double_is_upper_bound(f, b) {
        forall(i: Nat) {
            limit(f(i)) <= limit(f(i.suc))
            row_limit(f, i) <= row_limit(f, i.suc)
        }
        is_increasing(row_limit(f))
    }
}

/// The sequence of row limits is monotone.
theorem row_limits_is_monotone(f: (Nat, Nat) -> Real, b: Real) {
    doubly_increasing(f) and double_is_upper_bound(f, b)
    implies
    is_monotone(row_limit(f))
} by {
    if doubly_increasing(f) and double_is_upper_bound(f, b) {
        row_limits_is_increasing(f, b)
        is_increasing(row_limit(f))
        increasing_is_monotone(row_limit(f))
        is_monotone(row_limit(f))
    }
}

/// The sequence of column limits is increasing (full version).
theorem col_limits_is_increasing(f: (Nat, Nat) -> Real, b: Real) {
    doubly_increasing(f) and double_is_upper_bound(f, b)
    implies
    is_increasing(col_limit(f))
} by {
    if doubly_increasing(f) and double_is_upper_bound(f, b) {
        forall(j: Nat) {
            limit(flip(f, j)) <= limit(flip(f, j.suc))
            col_limit(f, j) <= col_limit(f, j.suc)
        }
        is_increasing(col_limit(f))
    }
}

/// The sequence of column limits is monotone.
theorem col_limits_is_monotone(f: (Nat, Nat) -> Real, b: Real) {
    doubly_increasing(f) and double_is_upper_bound(f, b)
    implies
    is_monotone(col_limit(f))
} by {
    if doubly_increasing(f) and double_is_upper_bound(f, b) {
        col_limits_is_increasing(f, b)
        is_increasing(col_limit(f))
        increasing_is_monotone(col_limit(f))
        is_monotone(col_limit(f))
    }
}

/// The sequence of row limits is bounded.
theorem row_limits_bounded(f: (Nat, Nat) -> Real, b: Real) {
    doubly_increasing(f) and double_is_upper_bound(f, b)
    implies
    is_upper_bound(row_limit(f), b)
} by {
    if doubly_increasing(f) and double_is_upper_bound(f, b) {
        forall(i: Nat) {
            limit(f(i)) <= b
            row_limit(f, i) <= b
        }
        is_upper_bound(row_limit(f), b)
    }
}

/// The sequence of column limits is bounded.
theorem col_limits_bounded(f: (Nat, Nat) -> Real, b: Real) {
    doubly_increasing(f) and double_is_upper_bound(f, b)
    implies
    is_upper_bound(col_limit(f), b)
} by {
    if doubly_increasing(f) and double_is_upper_bound(f, b) {
        forall(j: Nat) {
            limit(flip(f, j)) <= b
            col_limit(f, j) <= b
        }
        is_upper_bound(col_limit(f), b)
    }
}

/// The iterated limit over rows exists and converges.
theorem iterated_rows_converges(f: (Nat, Nat) -> Real, b: Real) {
    doubly_increasing(f) and double_is_upper_bound(f, b)
    implies
    converges(row_limit(f))
} by {
    if doubly_increasing(f) and double_is_upper_bound(f, b) {
        // row_limit(f) is increasing
        is_increasing(row_limit(f))

        // row_limit(f) is bounded by b
        is_upper_bound(row_limit(f), b)

        // Apply monotone convergence principle
        converges(row_limit(f))
    }
}

/// The iterated limit over rows exists by the shared monotone-map API.
theorem iterated_rows_converges_monotone(f: (Nat, Nat) -> Real, b: Real) {
    doubly_increasing(f) and double_is_upper_bound(f, b)
    implies
    converges(row_limit(f))
} by {
    if doubly_increasing(f) and double_is_upper_bound(f, b) {
        is_monotone(row_limit(f))
        is_upper_bound(row_limit(f), b)
        monotone_bounded_above_converges(row_limit(f), b)
        converges(row_limit(f))
    }
}

/// The iterated limit over columns exists and converges.
theorem iterated_cols_converges(f: (Nat, Nat) -> Real, b: Real) {
    doubly_increasing(f) and double_is_upper_bound(f, b)
    implies
    converges(col_limit(f))
} by {
    if doubly_increasing(f) and double_is_upper_bound(f, b) {
        // col_limit(f) is increasing
        is_increasing(col_limit(f))

        // col_limit(f) is bounded by b
        is_upper_bound(col_limit(f), b)

        // Apply monotone convergence principle
        converges(col_limit(f))
    }
}

/// The iterated limit over columns exists by the shared monotone-map API.
theorem iterated_cols_converges_monotone(f: (Nat, Nat) -> Real, b: Real) {
    doubly_increasing(f) and double_is_upper_bound(f, b)
    implies
    converges(col_limit(f))
} by {
    if doubly_increasing(f) and double_is_upper_bound(f, b) {
        is_monotone(col_limit(f))
        is_upper_bound(col_limit(f), b)
        monotone_bounded_above_converges(col_limit(f), b)
        converges(col_limit(f))
    }
}

/// The iterated limit over rows is an upper bound for f.
theorem iterated_limit_rows_is_upper_bound(f: (Nat, Nat) -> Real, b: Real) {
    doubly_increasing(f) and double_is_upper_bound(f, b)
    implies
    double_is_upper_bound(f, iterated_limit_rows(f))
} by {
    if doubly_increasing(f) and double_is_upper_bound(f, b) {
        forall(i: Nat, j: Nat) {
            // f(i) is increasing and converges, so f(i, j) <= limit(f(i))
            is_increasing(f(i))
            converges(f(i))
            is_upper_bound(f(i), limit(f(i)))
            f(i, j) <= limit(f(i))
            f(i, j) <= row_limit(f, i)

            // row_limit(f) is increasing and converges, so row_limit(f, i) <= limit(row_limit(f))
            is_increasing(row_limit(f))
            converges(row_limit(f))
            is_upper_bound(row_limit(f), limit(row_limit(f)))
            row_limit(f, i) <= limit(row_limit(f))
            row_limit(f, i) <= iterated_limit_rows(f)

            // By transitivity
            f(i, j) <= iterated_limit_rows(f)
        }
        double_is_upper_bound(f, iterated_limit_rows(f))
    }
}

/// The iterated limit over columns is an upper bound for f.
theorem iterated_limit_cols_is_upper_bound(f: (Nat, Nat) -> Real, b: Real) {
    doubly_increasing(f) and double_is_upper_bound(f, b)
    implies
    double_is_upper_bound(f, iterated_limit_cols(f))
} by {
    if doubly_increasing(f) and double_is_upper_bound(f, b) {
        forall(i: Nat, j: Nat) {
            // flip(f, j) is increasing and converges, so flip(f, j, i) = f(i, j) <= limit(flip(f, j))
            is_increasing(flip(f, j))
            converges(flip(f, j))
            is_upper_bound(flip(f, j), limit(flip(f, j)))
            flip(f, j, i) <= limit(flip(f, j))
            f(i, j) <= col_limit(f, j)

            // col_limit(f) is increasing and converges, so col_limit(f, j) <= limit(col_limit(f))
            is_increasing(col_limit(f))
            converges(col_limit(f))
            is_upper_bound(col_limit(f), limit(col_limit(f)))
            col_limit(f, j) <= limit(col_limit(f))
            col_limit(f, j) <= iterated_limit_cols(f)

            // By transitivity
            f(i, j) <= iterated_limit_cols(f)
        }
        double_is_upper_bound(f, iterated_limit_cols(f))
    }
}

/// If s is a supremum and b is an upper bound, then s <= b.
theorem supremum_lte_upper_bound(f: (Nat, Nat) -> Real, s: Real, b: Real) {
    double_image_is_supremum(f, s) and double_is_upper_bound(f, b)
    implies
    s <= b
} by {
    if double_image_is_supremum(f, s) and double_is_upper_bound(f, b) {
        // Proof by contradiction: assume s > b
        if s > b {
            // Then b < s, so by the supremum property, b is not an upper bound
            b < s
            not double_is_upper_bound(f, b)
            // But we assumed b is an upper bound, contradiction
            false
        }
        // Therefore s <= b
        s <= b
    }
}

/// If a double sequence converges to two values, they must be equal.
theorem double_converges_to_unique(f: (Nat, Nat) -> Real, a: Real, b: Real) {
    double_converges_to(f, a) and double_converges_to(f, b)
    implies
    a = b
} by {
    if double_converges_to(f, a) and double_converges_to(f, b) {
        // Proof by contradiction: assume a != b
        if a != b {
            // Then d = |a - b| > 0
            a - b != Real.0
            let d = (a - b).abs
            d.is_positive

            // Pick epsilon such that 0 < eps and eps + eps < d (i.e., eps < d/2)
            let eps: Real satisfy {
                eps.is_positive and eps + eps < d
            }

            // By convergence to a, there exists n1 such that for all i >= n1, j >= n1: |f(i,j) - a| < eps
            let n1: Nat satisfy {
                double_limit_condition(f, a, n1, eps)
            }

            // By convergence to b, there exists n2 such that for all i >= n2, j >= n2: |f(i,j) - b| < eps
            let n2: Nat satisfy {
                double_limit_condition(f, b, n2, eps)
            }

            // Take n = max(n1, n2)
            let n = n1.max(n2)
            n1 <= n
            n2 <= n

            // Then f(n, n) is within eps of both a and b
            double_limit_condition(f, a, n1, eps)
            f(n, n).is_close(a, eps)

            double_limit_condition(f, b, n2, eps)
            f(n, n).is_close(b, eps)

            // By triangle inequality: |a - b| <= |a - f(n,n)| + |f(n,n) - b| < eps + eps
            (a - b).abs < eps + eps

            // But we chose eps + eps < d = |a - b|, so |a - b| < eps + eps < |a - b|
            d < eps + eps
            (a - b).abs < (a - b).abs

            // This is a contradiction
            false
        }
        a = b
    }
}

/// If a double sequence converges to s, then its double limit equals s.
theorem double_limit_eq_of_converges_to(f: (Nat, Nat) -> Real, s: Real) {
    double_converges_to(f, s) implies double_limit(f) = s
} by {
    if double_converges_to(f, s) {
        // double_converges_to(f, s) implies double_converges(f)
        double_converges(f)

        // By the definition of double_limit, double_converges(f) implies double_converges_to(f, double_limit(f))
        double_converges_to(f, double_limit(f))

        // By uniqueness of limits
        s = double_limit(f)
        double_limit(f) = s
    }
}

/// For a doubly increasing function with supremum s, the iterated limit over rows equals s.
theorem iterated_limit_rows_eq_supremum(f: (Nat, Nat) -> Real, s: Real) {
    doubly_increasing(f) and double_image_is_supremum(f, s)
    implies
    iterated_limit_rows(f) = s
} by {
    if doubly_increasing(f) and double_image_is_supremum(f, s) {
        // s is an upper bound, so the iterated limit exists
        double_is_upper_bound(f, s)

        // Each f(i, j) <= row_limit(f, i) <= iterated_limit_rows(f)
        // So iterated_limit_rows(f) is an upper bound for the image
        // Since s is the least upper bound, s <= iterated_limit_rows(f)

        // Also, each row_limit(f, i) <= s, so iterated_limit_rows(f) <= s
        // Therefore iterated_limit_rows(f) = s

        // The row limits are bounded by s
        is_upper_bound(row_limit(f), s)
        forall(i: Nat) {
            row_limit(f, i) <= s
        }
        forall(i: Nat) {
            if Nat.zero <= i {
                row_limit(f, i) <= s
            }
        }
        exists(n: Nat) {
            forall(i: Nat) {
                n <= i implies row_limit(f, i) <= s
            }
        }
        eventual_ub(row_limit(f), s)

        // Apply the upper bound theorem to get iterated_limit_rows(f) <= s
        limit(row_limit(f)) <= s
        iterated_limit_rows(f) <= s

        // For the other direction: iterated_limit_rows(f) is an upper bound
        double_is_upper_bound(f, iterated_limit_rows(f))

        // Since s is the least upper bound, s <= iterated_limit_rows(f)
        s <= iterated_limit_rows(f)

        // Combining both directions
        iterated_limit_rows(f) = s
    }
}

/// For a doubly increasing function with supremum s, the iterated limit over columns equals s.
theorem iterated_limit_cols_eq_supremum(f: (Nat, Nat) -> Real, s: Real) {
    doubly_increasing(f) and double_image_is_supremum(f, s)
    implies
    iterated_limit_cols(f) = s
} by {
    if doubly_increasing(f) and double_image_is_supremum(f, s) {
        // s is an upper bound, so the iterated limit exists
        double_is_upper_bound(f, s)

        // The column limits are bounded by s
        is_upper_bound(col_limit(f), s)
        forall(j: Nat) {
            col_limit(f, j) <= s
        }
        forall(j: Nat) {
            if Nat.zero <= j {
                col_limit(f, j) <= s
            }
        }
        exists(n: Nat) {
            forall(j: Nat) {
                n <= j implies col_limit(f, j) <= s
            }
        }
        eventual_ub(col_limit(f), s)

        // Apply the upper bound theorem to get iterated_limit_cols(f) <= s
        limit(col_limit(f)) <= s
        iterated_limit_cols(f) <= s

        // For the other direction: iterated_limit_cols(f) is an upper bound
        double_is_upper_bound(f, iterated_limit_cols(f))

        // Since s is the least upper bound, s <= iterated_limit_cols(f)
        s <= iterated_limit_cols(f)

        // Combining both directions
        iterated_limit_cols(f) = s
    }
}

/// Monotone convergence allows interchange of limit order.
/// If f is doubly increasing and bounded, then all three limits exist and are equal:
/// lim_{i,j→∞} f(i,j) = lim_{i→∞} lim_{j→∞} f(i,j) = lim_{j→∞} lim_{i→∞} f(i,j).
theorem monotone_convergence_interchange(f: (Nat, Nat) -> Real, s: Real) {
    doubly_increasing(f) and double_image_is_supremum(f, s)
    implies
    double_limit(f) = s and
    iterated_limit_rows(f) = s and
    iterated_limit_cols(f) = s and
    double_limit(f) = iterated_limit_rows(f) and
    double_limit(f) = iterated_limit_cols(f) and
    iterated_limit_rows(f) = iterated_limit_cols(f)
} by {
    if doubly_increasing(f) and double_image_is_supremum(f, s) {
        // The double limit equals s
        double_converges_to(f, s)
        double_limit(f) = s

        // The iterated limits also equal s
        iterated_limit_rows(f) = s

        iterated_limit_cols(f) = s

        // Therefore all three are equal
        double_limit(f) = iterated_limit_rows(f)
        double_limit(f) = iterated_limit_cols(f)
        iterated_limit_rows(f) = iterated_limit_cols(f)

        // All conditions are satisfied
        double_limit(f) = s and
        iterated_limit_rows(f) = s and
        iterated_limit_cols(f) = s and
        double_limit(f) = iterated_limit_rows(f) and
        double_limit(f) = iterated_limit_cols(f) and
        iterated_limit_rows(f) = iterated_limit_cols(f)
    }
}

/// The double limit of the sum equals the sum of the double limits.
theorem double_limit_add(f: (Nat, Nat) -> Real, g: (Nat, Nat) -> Real) {
    double_converges(f) and double_converges(g)
    implies
    double_converges_to(add_fn_2(f, g), double_limit(f) + double_limit(g))
} by {
    if double_converges(f) and double_converges(g) {
        let lf = double_limit(f)
        let lg = double_limit(g)
        let sum_limit = lf + lg

        // f converges to lf and g converges to lg (by definition of double_limit)
        double_converges_to(f, lf)
        double_converges_to(g, lg)

        // Show add_fn_2(f, g) converges to lf + lg
        forall(eps: Real) {
            if eps.is_positive {
                // Split epsilon in half
                let eps2: Real satisfy {
                    eps2.is_positive and eps2 + eps2 < eps
                }

                // Get n1 where f is within eps2 of lf
                let n1: Nat satisfy {
                    double_limit_condition(f, lf, n1, eps2)
                }

                // Get n2 where g is within eps2 of lg
                let n2: Nat satisfy {
                    double_limit_condition(g, lg, n2, eps2)
                }

                // Take the max
                let n: Nat satisfy {
                    n1 <= n and n2 <= n
                }

                // Show add_fn_2(f, g) is within eps of sum_limit beyond n
                forall(i: Nat, j: Nat) {
                    if n <= i and n <= j {
                        n1 <= i and n1 <= j
                        double_limit_condition(f, lf, n1, eps2)
                        f(i, j).is_close(lf, eps2)

                        n2 <= i and n2 <= j
                        double_limit_condition(g, lg, n2, eps2)
                        g(i, j).is_close(lg, eps2)

                        add_fn_2(f, g, i, j) = f(i, j) + g(i, j)

                        // Both f and g are close to their limits
                        f(i, j).is_close(lf, eps2)
                        g(i, j).is_close(lg, eps2)

                        // Therefore their sum is close to the sum of limits
                        (f(i, j) + g(i, j)).is_close(lf + lg, eps2 + eps2)
                        eps2 + eps2 < eps
                        (f(i, j) + g(i, j)).is_close(lf + lg, eps)
                        add_fn_2(f, g, i, j).is_close(sum_limit, eps)
                    }
                }
                exists(k0: Nat, k1: Nat) {
                    n <= k0 and n <= k1 and not add_fn_2(f, g, k0, k1).is_close(sum_limit, eps)
                } or double_limit_condition(add_fn_2(f, g), sum_limit, n, eps)
                double_limit_condition(add_fn_2(f, g), sum_limit, n, eps)
            }
        }
        double_converges_to(add_fn_2(f, g), sum_limit)
        double_converges_to(add_fn_2(f, g), lf + lg)
        double_converges_to(add_fn_2(f, g), double_limit(f) + double_limit(g))
    }
}

/// The double limit of the difference equals the difference of the double limits.
theorem double_limit_sub(f: (Nat, Nat) -> Real, g: (Nat, Nat) -> Real) {
    double_converges(f) and double_converges(g)
    implies
    double_converges_to(sub_fn_2(f, g), double_limit(f) - double_limit(g))
} by {
    if double_converges(f) and double_converges(g) {
        let lf = double_limit(f)
        let lg = double_limit(g)
        let diff_limit = lf - lg

        // f converges to lf and g converges to lg (by definition of double_limit)
        double_converges_to(f, lf)
        double_converges_to(g, lg)

        // Show sub_fn_2(f, g) converges to lf - lg
        forall(eps: Real) {
            if eps.is_positive {
                // Split epsilon in half
                let eps2: Real satisfy {
                    eps2.is_positive and eps2 + eps2 < eps
                }

                // Get n1 where f is within eps2 of lf
                let n1: Nat satisfy {
                    double_limit_condition(f, lf, n1, eps2)
                }

                // Get n2 where g is within eps2 of lg
                let n2: Nat satisfy {
                    double_limit_condition(g, lg, n2, eps2)
                }

                // Take the max
                let n: Nat satisfy {
                    n1 <= n and n2 <= n
                }

                // Show sub_fn_2(f, g) is within eps of diff_limit beyond n
                forall(i: Nat, j: Nat) {
                    if n <= i and n <= j {
                        n1 <= i and n1 <= j
                        double_limit_condition(f, lf, n1, eps2)
                        f(i, j).is_close(lf, eps2)

                        n2 <= i and n2 <= j
                        double_limit_condition(g, lg, n2, eps2)
                        g(i, j).is_close(lg, eps2)

                        sub_fn_2(f, g, i, j) = f(i, j) - g(i, j)

                        // Both f and g are close to their limits
                        f(i, j).is_close(lf, eps2)
                        g(i, j).is_close(lg, eps2)

                        // Express subtraction as addition with negation
                        let neg_g = Real.0 - g(i, j)
                        let neg_lg = Real.0 - lg
                        f(i, j) - g(i, j) = f(i, j) + neg_g
                        lf - lg = lf + neg_lg

                        // g close to lg means -g close to -lg
                        // |g - lg| < eps2, so |-g - (-lg)| = |-(g - lg)| = |g - lg| < eps2
                        g(i, j).is_close(lg, eps2)
                        (g(i, j) - lg).abs < eps2
                        neg_g - neg_lg = Real.0 - g(i, j) - (Real.0 - lg)
                        neg_g - neg_lg = Real.0 - (Real.0 - lg) - g(i, j)
                        neg_g - neg_lg = lg - g(i, j)
                        neg_g - neg_lg = Real.0 - (g(i, j) - lg)
                        (neg_g - neg_lg).abs = (Real.0 - (g(i, j) - lg)).abs
                        (Real.0 - (g(i, j) - lg)).abs = (g(i, j) - lg).abs
                        (neg_g - neg_lg).abs < eps2
                        neg_g.is_close(neg_lg, eps2)

                        // Sum of close values is close
                        (f(i, j) + neg_g).is_close(lf + neg_lg, eps2 + eps2)
                        (f(i, j) - g(i, j)).is_close(lf - lg, eps2 + eps2)
                        eps2 + eps2 < eps
                        (f(i, j) - g(i, j)).is_close(lf - lg, eps)
                        sub_fn_2(f, g, i, j).is_close(diff_limit, eps)
                    }
                }
                exists(k0: Nat, k1: Nat) {
                    n <= k0 and n <= k1 and not sub_fn_2(f, g, k0, k1).is_close(diff_limit, eps)
                } or double_limit_condition(sub_fn_2(f, g), diff_limit, n, eps)
                double_limit_condition(sub_fn_2(f, g), diff_limit, n, eps)
            }
        }
        double_converges_to(sub_fn_2(f, g), diff_limit)
        double_converges_to(sub_fn_2(f, g), lf - lg)
        double_converges_to(sub_fn_2(f, g), double_limit(f) - double_limit(g))
    }
}
