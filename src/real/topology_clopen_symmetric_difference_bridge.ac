/// Clopen real-set symmetric difference bridge.

from data.basic.set import Set, difference_contains_eq, union_comm, union_contains_eq
from real.real_field import Real
from real.topology import is_closed_set, is_open_set
from real.topology_closed_open import union_of_open_is_open
from real.topology_clopen import clopen_real_set_intro, is_clopen_real_set
from real.topology_clopen_difference_bridge import difference_of_clopen_real_sets_is_closed,
    difference_of_clopen_real_sets_is_clopen, difference_of_clopen_real_sets_is_open
from real.topology_closure_union import union_of_closed_is_closed

/// Membership in the symmetric difference of two real sets is exclusive membership.
theorem clopen_real_sets_difference_union_contains_eq(s: Set[Real], t: Set[Real], x: Real) {
    s.difference(t).union(t.difference(s)).contains(x) =
        ((s.contains(x) and not t.contains(x)) or (t.contains(x) and not s.contains(x)))
} by {
    union_contains_eq[Real](s.difference(t), t.difference(s), x)
    difference_contains_eq[Real](s, t, x)
    difference_contains_eq[Real](t, s, x)
}

/// The symmetric difference of two clopen real sets is open.
theorem symmetric_difference_of_clopen_real_sets_is_open(s: Set[Real], t: Set[Real]) {
    is_clopen_real_set(s) and is_clopen_real_set(t) implies
        is_open_set(s.difference(t).union(t.difference(s)))
} by {
    if is_clopen_real_set(s) and is_clopen_real_set(t) {
        difference_of_clopen_real_sets_is_open(s, t)
        difference_of_clopen_real_sets_is_open(t, s)
        is_open_set(s.difference(t))
        is_open_set(t.difference(s))
        union_of_open_is_open(s.difference(t), t.difference(s))
        is_open_set(s.difference(t).union(t.difference(s)))
    }
}

/// The symmetric difference of two clopen real sets is closed.
theorem symmetric_difference_of_clopen_real_sets_is_closed(s: Set[Real], t: Set[Real]) {
    is_clopen_real_set(s) and is_clopen_real_set(t) implies
        is_closed_set(s.difference(t).union(t.difference(s)))
} by {
    if is_clopen_real_set(s) and is_clopen_real_set(t) {
        difference_of_clopen_real_sets_is_closed(s, t)
        difference_of_clopen_real_sets_is_closed(t, s)
        is_closed_set(s.difference(t))
        is_closed_set(t.difference(s))
        union_of_closed_is_closed(s.difference(t), t.difference(s))
        is_closed_set(s.difference(t).union(t.difference(s)))
    }
}

/// The symmetric difference of two clopen real sets is clopen.
theorem symmetric_difference_of_clopen_real_sets_is_clopen(s: Set[Real], t: Set[Real]) {
    is_clopen_real_set(s) and is_clopen_real_set(t) implies
        is_clopen_real_set(s.difference(t).union(t.difference(s)))
} by {
    if is_clopen_real_set(s) and is_clopen_real_set(t) {
        difference_of_clopen_real_sets_is_clopen(s, t)
        difference_of_clopen_real_sets_is_clopen(t, s)
        is_clopen_real_set(s.difference(t))
        is_clopen_real_set(t.difference(s))
        symmetric_difference_of_clopen_real_sets_is_open(s, t)
        symmetric_difference_of_clopen_real_sets_is_closed(s, t)
        is_open_set(s.difference(t).union(t.difference(s)))
        is_closed_set(s.difference(t).union(t.difference(s)))
        clopen_real_set_intro(s.difference(t).union(t.difference(s)))
        is_clopen_real_set(s.difference(t).union(t.difference(s)))
    }
}

/// Symmetric difference of real sets is commutative.
theorem clopen_real_sets_symmetric_difference_comm(s: Set[Real], t: Set[Real]) {
    s.difference(t).union(t.difference(s)) = t.difference(s).union(s.difference(t))
} by {
    union_comm[Real](s.difference(t), t.difference(s))
}
