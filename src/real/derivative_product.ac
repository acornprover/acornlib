from data.basic.function_algebra import pointwise_mul
from real.continuity_base import continuous_at, continuous_condition
from real.derivative_basic import difference_quotient, has_derivative_at,
    differentiable_at, sub_ne_zero_of_ne
from real.derivative_continuity import derivative_continuous_at
from real.prod_seq import abs_le_abs_add_eps, mul_close_from_close
from real.real_base import Real, abs_gte_zero, abs_not_neg, lte_lt_trans,
    lt_add_pos, lt_trans
from real.real_ring import exists_small_mul_variant_2, mul_abs
from real.real_seq import add_close, close_and_lt_imp_close,
    eps_smaller_than_both

/// Equality on the left of a closeness relation may be transported across the relation.
theorem close_eq_left(a: Real, b: Real, c: Real, eps: Real) {
    a = b and a.is_close(c, eps) implies b.is_close(c, eps)
} by {
    b = a
}

/// The absolute-value bound one larger than a real absolute value is positive.
theorem abs_add_one_pos(a: Real) {
    (a.abs + Real.1).is_positive
} by {
    abs_gte_zero(a)
    a.abs >= Real.0
    a.abs < a.abs + Real.1
    lte_lt_trans(Real.0, a.abs, a.abs + Real.1)
    Real.0 < a.abs + Real.1
}

/// The absolute value of a real is bounded by one plus that absolute value.
theorem abs_lte_abs_add_one(a: Real) {
    a.abs <= a.abs + Real.1
} by {
    a.abs < a.abs + Real.1
}

/// Subtracting through an intermediate value telescopes.
theorem sub_sub_telescope(a: Real, b: Real, c: Real) {
    a - b + (b - c) = a - c
} by {
    a - b = a + -b
    b - c = b + -c
    a - b + (b - c) = a + -b + (b + -c)
    a + -b + (b + -c) = a + (-b + b) + -c
    -b + b = Real.0
    a + (-b + b) + -c = a + Real.0 + -c
    a + Real.0 = a
    a + Real.0 + -c = a + -c
    a + -c = a - c
}

/// A fixed scalar preserves closeness with tolerance multiplied by one plus its absolute value.
theorem scalar_close_wide(c: Real, a: Real, b: Real, eps: Real) {
    a.is_close(b, eps) and eps.is_positive
    implies (c * a).is_close(c * b, (c.abs + Real.1) * eps)
} by {
    a.is_close(b, eps) = (a - b).abs < eps
    (a - b).abs < eps
    c * a - c * b = c * (a - b)
    mul_abs(c, a - b)
    (c * (a - b)).abs = c.abs * (a - b).abs
    (c * a - c * b).abs = c.abs * (a - b).abs
    abs_not_neg(c)
    not c.abs.is_negative
    if c.abs = Real.0 {
        c.abs * (a - b).abs = Real.0
        Real.0 < eps
        Real.1 * eps = eps
        c.abs + Real.1 = Real.1
        (c.abs + Real.1) * eps = eps
        (c * a - c * b).abs < (c.abs + Real.1) * eps
    } else {
        c.abs.is_positive
        c.abs * (a - b).abs < c.abs * eps
        Real.1 * eps = eps
        c.abs * eps < c.abs * eps + eps
        c.abs * eps + eps = c.abs * eps + Real.1 * eps
        c.abs * eps + Real.1 * eps = (c.abs + Real.1) * eps
        c.abs * eps < (c.abs + Real.1) * eps
        lt_trans(c.abs * (a - b).abs, c.abs * eps, (c.abs + Real.1) * eps)
        c.abs * (a - b).abs < (c.abs + Real.1) * eps
    }
    (c * a - c * b).abs < (c.abs + Real.1) * eps
    (c * a).is_close(c * b, (c.abs + Real.1) * eps)
}

/// The product difference splits into one increment of each factor.
theorem product_diff_split(fx: Real, gx: Real, fx0: Real, gx0: Real) {
    fx * gx - fx0 * gx0 = fx * (gx - gx0) + gx0 * (fx - fx0)
} by {
    fx * (gx - gx0) = fx * gx - fx * gx0
    gx0 * (fx - fx0) = gx0 * fx - gx0 * fx0
    gx0 * fx = fx * gx0
    gx0 * fx - gx0 * fx0 = fx * gx0 - gx0 * fx0
    fx * (gx - gx0) + gx0 * (fx - fx0) =
        fx * gx - fx * gx0 + (fx * gx0 - gx0 * fx0)
    sub_sub_telescope(fx * gx, fx * gx0, gx0 * fx0)
    fx * gx - fx * gx0 + (fx * gx0 - gx0 * fx0) = fx * gx - gx0 * fx0
    fx0 * gx0 = gx0 * fx0
}

/// A scalar factor in the numerator may be pulled outside a quotient.
theorem scalar_div(c: Real, a: Real, b: Real) {
    (c * a) / b = c * (a / b)
} by {
    (c * a) / b = (c * a) * b.inverse
    a / b = a * b.inverse
    c * (a / b) = c * (a * b.inverse)
    (c * a) * b.inverse = c * (a * b.inverse)
}

/// Division distributes over addition when the denominator is nonzero.
theorem div_add_distrib(a: Real, b: Real, c: Real) {
    c != Real.0 implies (a + b) / c = a / c + b / c
} by {
    if c != Real.0 {
        (a + b) * c.inverse = a * c.inverse + b * c.inverse
    }
}

/// The difference quotient of a pointwise product has the product-rule split form.
theorem difference_quotient_pointwise_mul(f: Real -> Real, g: Real -> Real, x0: Real, x: Real) {
    x != x0 implies difference_quotient(pointwise_mul(f, g), x0, x) =
        f(x) * difference_quotient(g, x0, x) + g(x0) * difference_quotient(f, x0, x)
} by {
    if x != x0 {
        sub_ne_zero_of_ne(x, x0)
        x - x0 != Real.0
        pointwise_mul(f, g, x) = f(x) * g(x)
        pointwise_mul(f, g, x0) = f(x0) * g(x0)
        product_diff_split(f(x), g(x), f(x0), g(x0))
        let numer = f(x) * g(x) - f(x0) * g(x0)
        numer = f(x) * (g(x) - g(x0)) + g(x0) * (f(x) - f(x0))
        let p = f(x) * (g(x) - g(x0))
        let q = g(x0) * (f(x) - f(x0))
        numer = p + q
        div_add_distrib(p, q, x - x0)
        (p + q) / (x - x0) = p / (x - x0) + q / (x - x0)
        scalar_div(f(x), g(x) - g(x0), x - x0)
        p / (x - x0) = f(x) * ((g(x) - g(x0)) / (x - x0))
        scalar_div(g(x0), f(x) - f(x0), x - x0)
        q / (x - x0) = g(x0) * ((f(x) - f(x0)) / (x - x0))
        numer / (x - x0) =
            f(x) * ((g(x) - g(x0)) / (x - x0)) +
            g(x0) * ((f(x) - f(x0)) / (x - x0))
        difference_quotient(pointwise_mul(f, g), x0, x) = numer / (x - x0)
        difference_quotient(g, x0, x) = (g(x) - g(x0)) / (x - x0)
        difference_quotient(f, x0, x) = (f(x) - f(x0)) / (x - x0)
        difference_quotient(pointwise_mul(f, g), x0, x) =
            f(x) * difference_quotient(g, x0, x) + g(x0) * difference_quotient(f, x0, x)
    }
}

/// Difference quotients in the product-rule split form inherit the desired derivative limit.
theorem product_split_difference_quotient_close(
    f: Real -> Real, g: Real -> Real, x0: Real, x: Real,
    df: Real, dg: Real,
    f_bound: Real, dg_bound: Real, g_bound: Real,
    eps_small: Real, eps: Real
) {
    x != x0 and
    f(x).is_close(f(x0), eps_small) and
    difference_quotient(g, x0, x).is_close(dg, eps_small) and
    difference_quotient(f, x0, x).is_close(df, eps_small) and
    f(x).abs <= f_bound and dg.abs <= dg_bound and
    f_bound.is_positive and dg_bound.is_positive and g_bound = g(x0).abs + Real.1 and
    eps_small.is_positive and
    (f_bound * eps_small + eps_small * dg_bound) + g_bound * eps_small < eps
    implies
    (f(x) * difference_quotient(g, x0, x) + g(x0) * difference_quotient(f, x0, x)).is_close(
        f(x0) * dg + g(x0) * df,
        eps
    )
} by {
    if x != x0 and
       f(x).is_close(f(x0), eps_small) and
       difference_quotient(g, x0, x).is_close(dg, eps_small) and
       difference_quotient(f, x0, x).is_close(df, eps_small) and
       f(x).abs <= f_bound and dg.abs <= dg_bound and
       f_bound.is_positive and dg_bound.is_positive and g_bound = g(x0).abs + Real.1 and
       eps_small.is_positive and
       (f_bound * eps_small + eps_small * dg_bound) + g_bound * eps_small < eps {
        mul_close_from_close(f(x), f(x0), difference_quotient(g, x0, x), dg,
            eps_small, eps_small, f_bound, dg_bound)
        (f(x) * difference_quotient(g, x0, x)).is_close(
            f(x0) * dg,
            f_bound * eps_small + eps_small * dg_bound
        )
        scalar_close_wide(g(x0), difference_quotient(f, x0, x), df, eps_small)
        (g(x0) * difference_quotient(f, x0, x)).is_close(
            g(x0) * df,
            (g(x0).abs + Real.1) * eps_small
        )
        g_bound = g(x0).abs + Real.1
        (g(x0).abs + Real.1) * eps_small = g_bound * eps_small
        (g(x0) * difference_quotient(f, x0, x)).is_close(
            g(x0) * df,
            g_bound * eps_small
        )
        add_close(
            f(x) * difference_quotient(g, x0, x),
            g(x0) * difference_quotient(f, x0, x),
            f(x0) * dg,
            g(x0) * df,
            f_bound * eps_small + eps_small * dg_bound,
            g_bound * eps_small
        )
        (f(x) * difference_quotient(g, x0, x) + g(x0) * difference_quotient(f, x0, x)).is_close(
            f(x0) * dg + g(x0) * df,
            (f_bound * eps_small + eps_small * dg_bound) + g_bound * eps_small
        )
        close_and_lt_imp_close(
            f(x) * difference_quotient(g, x0, x) + g(x0) * difference_quotient(f, x0, x),
            f(x0) * dg + g(x0) * df,
            (f_bound * eps_small + eps_small * dg_bound) + g_bound * eps_small,
            eps
        )
        (f(x) * difference_quotient(g, x0, x) + g(x0) * difference_quotient(f, x0, x)).is_close(
            f(x0) * dg + g(x0) * df,
            eps
        )
    }
}

/// Pointwise products satisfy the product rule for derivatives.
theorem derivative_pointwise_mul(f: Real -> Real, g: Real -> Real, x0: Real, df: Real, dg: Real) {
    has_derivative_at(f, x0, df) and has_derivative_at(g, x0, dg)
    implies has_derivative_at(pointwise_mul(f, g), x0, f(x0) * dg + g(x0) * df)
} by {
    derivative_continuous_at(f, x0, df)
    continuous_at(f, x0)
    continuous_at(f, x0) = forall(eps: Real) {
        eps.is_positive implies exists(delta: Real) {
            delta.is_positive and continuous_condition(f, x0, delta, eps)
        }
    }
    has_derivative_at(f, x0, df) = forall(eps: Real) {
        eps.is_positive implies exists(delta: Real) {
            delta.is_positive and forall(x: Real) {
                x != x0 and x.is_close(x0, delta)
                implies difference_quotient(f, x0, x).is_close(df, eps)
            }
        }
    }
    has_derivative_at(g, x0, dg) = forall(eps: Real) {
        eps.is_positive implies exists(delta: Real) {
            delta.is_positive and forall(x: Real) {
                x != x0 and x.is_close(x0, delta)
                implies difference_quotient(g, x0, x).is_close(dg, eps)
            }
        }
    }
    forall(eps: Real) {
        if eps.is_positive {
            let f_bound = f(x0).abs + Real.1
            abs_add_one_pos(f(x0))
            f_bound.is_positive
            let dg_bound = dg.abs + Real.1
            abs_add_one_pos(dg)
            dg_bound.is_positive
            let g_bound = g(x0).abs + Real.1
            abs_add_one_pos(g(x0))
            g_bound.is_positive
            let big_bound = f_bound + dg_bound + g_bound
            f_bound > Real.0
            dg_bound > Real.0
            g_bound > Real.0
            lt_add_pos(f_bound, dg_bound)
            f_bound < f_bound + dg_bound
            lt_trans(Real.0, f_bound, f_bound + dg_bound)
            f_bound + dg_bound > Real.0
            Real.0 < f_bound + dg_bound
            lt_add_pos(f_bound + dg_bound, g_bound)
            f_bound + dg_bound < (f_bound + dg_bound) + g_bound
            f_bound + dg_bound + g_bound = (f_bound + dg_bound) + g_bound
            lt_trans(Real.0, f_bound + dg_bound, (f_bound + dg_bound) + g_bound)
            Real.0 < (f_bound + dg_bound) + g_bound
            (f_bound + dg_bound) + g_bound > Real.0
            big_bound = (f_bound + dg_bound) + g_bound
            big_bound > Real.0
            big_bound.is_positive
            exists_small_mul_variant_2(big_bound, eps)
            let eps_small: Real satisfy {
                eps_small.is_positive and eps_small * big_bound < eps
            }
            let delta_cont: Real satisfy {
                delta_cont.is_positive and continuous_condition(f, x0, delta_cont, eps_small)
            }
            Real.1.is_positive
            let delta_bound: Real satisfy {
                delta_bound.is_positive and continuous_condition(f, x0, delta_bound, Real.1)
            }
            let delta_f: Real satisfy {
                delta_f.is_positive and forall(x: Real) {
                    x != x0 and x.is_close(x0, delta_f)
                    implies difference_quotient(f, x0, x).is_close(df, eps_small)
                }
            }
            let delta_g: Real satisfy {
                delta_g.is_positive and forall(x: Real) {
                    x != x0 and x.is_close(x0, delta_g)
                    implies difference_quotient(g, x0, x).is_close(dg, eps_small)
                }
            }
            eps_smaller_than_both(delta_cont, delta_bound)
            let delta_12: Real satisfy {
                delta_12.is_positive and delta_12 < delta_cont and delta_12 < delta_bound
            }
            eps_smaller_than_both(delta_f, delta_g)
            let delta_34: Real satisfy {
                delta_34.is_positive and delta_34 < delta_f and delta_34 < delta_g
            }
            eps_smaller_than_both(delta_12, delta_34)
            let delta: Real satisfy {
                delta.is_positive and delta < delta_12 and delta < delta_34
            }
            forall(x: Real) {
                if x != x0 and x.is_close(x0, delta) {
                    close_and_lt_imp_close(x, x0, delta, delta_12)
                    close_and_lt_imp_close(x, x0, delta, delta_34)
                    x.is_close(x0, delta_12)
                    x.is_close(x0, delta_34)
                    close_and_lt_imp_close(x, x0, delta_12, delta_cont)
                    close_and_lt_imp_close(x, x0, delta_12, delta_bound)
                    close_and_lt_imp_close(x, x0, delta_34, delta_f)
                    close_and_lt_imp_close(x, x0, delta_34, delta_g)
                    x.is_close(x0, delta_cont)
                    x.is_close(x0, delta_bound)
                    x.is_close(x0, delta_f)
                    x.is_close(x0, delta_g)
                    continuous_condition(f, x0, delta_cont, eps_small) = forall(y: Real) {
                        y.is_close(x0, delta_cont) implies f(y).is_close(f(x0), eps_small)
                    }
                    continuous_condition(f, x0, delta_bound, Real.1) = forall(y: Real) {
                        y.is_close(x0, delta_bound) implies f(y).is_close(f(x0), Real.1)
                    }
                    f(x).is_close(f(x0), eps_small)
                    f(x).is_close(f(x0), Real.1)
                    abs_le_abs_add_eps(f(x), f(x0), Real.1)
                    f(x).abs <= f(x0).abs + Real.1
                    f(x).abs <= f_bound
                    x != x0 and x.is_close(x0, delta_f)
                    x != x0 and x.is_close(x0, delta_g)
                    if x != x0 and x.is_close(x0, delta_f) {
                        difference_quotient(f, x0, x).is_close(df, eps_small)
                    }
                    if x != x0 and x.is_close(x0, delta_g) {
                        difference_quotient(g, x0, x).is_close(dg, eps_small)
                    }
                    difference_quotient(f, x0, x).is_close(df, eps_small)
                    difference_quotient(g, x0, x).is_close(dg, eps_small)
                    abs_lte_abs_add_one(dg)
                    dg.abs <= dg_bound
                    let total_eps = (f_bound * eps_small + eps_small * dg_bound) + g_bound * eps_small
                    f_bound * eps_small = eps_small * f_bound
                    g_bound * eps_small = eps_small * g_bound
                    eps_small * f_bound + eps_small * dg_bound = eps_small * (f_bound + dg_bound)
                    total_eps = eps_small * (f_bound + dg_bound) + eps_small * g_bound
                    eps_small * (f_bound + dg_bound) + eps_small * g_bound =
                        eps_small * (f_bound + dg_bound + g_bound)
                    total_eps = eps_small * big_bound
                    total_eps < eps
                    product_split_difference_quotient_close(
                        f, g, x0, x, df, dg, f_bound, dg_bound, g_bound, eps_small, eps
                    )
                    (f(x) * difference_quotient(g, x0, x) + g(x0) * difference_quotient(f, x0, x)).is_close(
                        f(x0) * dg + g(x0) * df,
                        eps
                    )
                    difference_quotient_pointwise_mul(f, g, x0, x)
                    difference_quotient(pointwise_mul(f, g), x0, x) =
                        f(x) * difference_quotient(g, x0, x) + g(x0) * difference_quotient(f, x0, x)
                    close_eq_left(
                        f(x) * difference_quotient(g, x0, x) + g(x0) * difference_quotient(f, x0, x),
                        difference_quotient(pointwise_mul(f, g), x0, x),
                        f(x0) * dg + g(x0) * df,
                        eps
                    )
                    difference_quotient(pointwise_mul(f, g), x0, x).is_close(f(x0) * dg + g(x0) * df, eps)
                }
            }
            delta.is_positive and forall(x: Real) {
                x != x0 and x.is_close(x0, delta)
                implies difference_quotient(pointwise_mul(f, g), x0, x).is_close(f(x0) * dg + g(x0) * df, eps)
            }
            exists(delta2: Real) {
                delta2.is_positive and forall(x: Real) {
                    x != x0 and x.is_close(x0, delta2)
                    implies difference_quotient(pointwise_mul(f, g), x0, x).is_close(f(x0) * dg + g(x0) * df, eps)
                }
            }
        }
    }
    has_derivative_at(pointwise_mul(f, g), x0, f(x0) * dg + g(x0) * df) = forall(eps: Real) {
        eps.is_positive implies exists(delta: Real) {
            delta.is_positive and forall(x: Real) {
                x != x0 and x.is_close(x0, delta)
                implies difference_quotient(pointwise_mul(f, g), x0, x).is_close(f(x0) * dg + g(x0) * df, eps)
            }
        }
    }
    if not has_derivative_at(pointwise_mul(f, g), x0, f(x0) * dg + g(x0) * df) {
        not forall(eps: Real) {
            eps.is_positive implies exists(delta: Real) {
                delta.is_positive and forall(x: Real) {
                    x != x0 and x.is_close(x0, delta)
                    implies difference_quotient(pointwise_mul(f, g), x0, x).is_close(f(x0) * dg + g(x0) * df, eps)
                }
            }
        }
        let bad_eps: Real satisfy {
            bad_eps.is_positive and forall(delta: Real) {
                not (delta.is_positive and forall(x: Real) {
                    x != x0 and x.is_close(x0, delta)
                    implies difference_quotient(pointwise_mul(f, g), x0, x).is_close(f(x0) * dg + g(x0) * df, bad_eps)
                })
            }
        }
        exists(delta: Real) {
            delta.is_positive and forall(x: Real) {
                x != x0 and x.is_close(x0, delta)
                implies difference_quotient(pointwise_mul(f, g), x0, x).is_close(f(x0) * dg + g(x0) * df, bad_eps)
            }
        }
        false
    }
}

/// Differentiability is preserved by pointwise products.
theorem differentiable_pointwise_mul(f: Real -> Real, g: Real -> Real, x0: Real) {
    differentiable_at(f, x0) and differentiable_at(g, x0)
    implies differentiable_at(pointwise_mul(f, g), x0)
} by {
    let df: Real satisfy {
        has_derivative_at(f, x0, df)
    }
    let dg: Real satisfy {
        has_derivative_at(g, x0, dg)
    }
    derivative_pointwise_mul(f, g, x0, df, dg)
    has_derivative_at(pointwise_mul(f, g), x0, f(x0) * dg + g(x0) * df)
    exists(d: Real) {
        has_derivative_at(pointwise_mul(f, g), x0, d)
    }
}

/// The derivative of the pointwise square is the product rule with equal factors.
theorem derivative_pointwise_square(f: Real -> Real, x0: Real, d: Real) {
    has_derivative_at(f, x0, d)
    implies has_derivative_at(pointwise_mul(f, f), x0, f(x0) * d + f(x0) * d)
} by {
    derivative_pointwise_mul(f, f, x0, d, d)
    has_derivative_at(pointwise_mul(f, f), x0, f(x0) * d + f(x0) * d)
}

/// Differentiability is preserved by pointwise squaring.
theorem differentiable_pointwise_square(f: Real -> Real, x0: Real) {
    differentiable_at(f, x0) implies differentiable_at(pointwise_mul(f, f), x0)
} by {
    differentiable_pointwise_mul(f, f, x0)
    differentiable_at(pointwise_mul(f, f), x0)
}
