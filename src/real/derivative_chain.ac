from data.basic.functions import compose
from real.continuity_base import continuous_at, continuous_condition
from real.derivative_basic import difference_quotient, has_derivative_at,
    differentiable_at, sub_ne_zero_of_ne
from real.derivative_continuity import derivative_continuous_at
from real.derivative_product import abs_add_one_pos, abs_lte_abs_add_one,
    close_eq_left
from real.prod_seq import abs_le_abs_add_eps, mul_close_from_close
from real.real_base import Real, self_close
from real.real_field import div_mul_cancel_left
from real.real_ring import exists_small_mul_variant_2, mul_zero_left
from real.real_seq import close_and_lt_imp_close, eps_smaller_than_both

/// The chain-rule secant factor, completed at the base value by the outer derivative.
define chain_difference_factor(g: Real -> Real, y0: Real, dg: Real, y: Real) -> Real {
    if y = y0 {
        dg
    } else {
        difference_quotient(g, y0, y)
    }
}

/// At the base value, the completed chain-rule secant factor is the outer derivative.
theorem chain_difference_factor_at_base(g: Real -> Real, y0: Real, dg: Real) {
    chain_difference_factor(g, y0, dg, y0) = dg
}

/// Away from the base value, the completed chain-rule secant factor is the ordinary difference quotient.
theorem chain_difference_factor_off_base(g: Real -> Real, y0: Real, dg: Real, y: Real) {
    y != y0 implies chain_difference_factor(g, y0, dg, y) = difference_quotient(g, y0, y)
}

/// A zero function increment has zero difference quotient away from the base point.
theorem difference_quotient_eq_zero_of_value_eq(f: Real -> Real, x0: Real, x: Real) {
    x != x0 and f(x) = f(x0) implies difference_quotient(f, x0, x) = Real.0
} by {
    f(x) - f(x0) = Real.0
    difference_quotient(f, x0, x) = Real.0 / (x - x0)
    Real.0 / (x - x0) = Real.0 * (x - x0).inverse
    mul_zero_left((x - x0).inverse)
    Real.0 / (x - x0) = Real.0
}

/// A local derivative closeness bound applies at any point satisfying its hypotheses.
theorem derivative_close_apply(f: Real -> Real, x0: Real, d: Real, delta: Real, eps: Real, x: Real) {
    (forall(y: Real) {
        y != x0 and y.is_close(x0, delta)
        implies difference_quotient(f, x0, y).is_close(d, eps)
    }) and x != x0 and x.is_close(x0, delta)
    implies difference_quotient(f, x0, x).is_close(d, eps)
} by {
    if x != x0 and x.is_close(x0, delta) {
        difference_quotient(f, x0, x).is_close(d, eps)
    }
}

/// Multiplying two successive nonzero difference quotients collapses to the composite difference quotient.
theorem difference_quotient_compose_nonzero_inner(
    f: Real -> Real, g: Real -> Real, x0: Real, x: Real
) {
    x != x0 and f(x) != f(x0) implies
    difference_quotient(g, f(x0), f(x)) * difference_quotient(f, x0, x) =
        difference_quotient(compose(g, f), x0, x)
} by {
    if x != x0 and f(x) != f(x0) {
        sub_ne_zero_of_ne(x, x0)
        sub_ne_zero_of_ne(f(x), f(x0))
        let outer_num = g(f(x)) - g(f(x0))
        let inner_num = f(x) - f(x0)
        let outer_den = f(x) - f(x0)
        let inner_den = x - x0
        outer_den != Real.0
        inner_den != Real.0
        outer_den = inner_num
        difference_quotient(g, f(x0), f(x)) = outer_num / outer_den
        difference_quotient(f, x0, x) = inner_num / inner_den
        (outer_num / outer_den) * (inner_num / inner_den) =
            (outer_num / outer_den) * (outer_den / inner_den)
        div_mul_cancel_left(outer_den, outer_num)
        (outer_den * (outer_num / outer_den)) / outer_den = outer_num / outer_den
        (outer_num / outer_den) * (outer_den / inner_den) = outer_num / inner_den
        compose(g, f, x) = g(f(x))
        compose(g, f, x0) = g(f(x0))
        difference_quotient(compose(g, f), x0, x) = outer_num / inner_den
        difference_quotient(g, f(x0), f(x)) * difference_quotient(f, x0, x) =
            difference_quotient(compose(g, f), x0, x)
    }
}

/// The completed chain-rule factor gives an exact factorization of the composite difference quotient.
theorem difference_quotient_compose_factor(
    f: Real -> Real, g: Real -> Real, x0: Real, x: Real, dg: Real
) {
    x != x0 implies
    difference_quotient(compose(g, f), x0, x) =
        chain_difference_factor(g, f(x0), dg, f(x)) * difference_quotient(f, x0, x)
} by {
    if x != x0 {
        if f(x) = f(x0) {
            chain_difference_factor_at_base(g, f(x0), dg)
            chain_difference_factor(g, f(x0), dg, f(x)) = dg
            difference_quotient_eq_zero_of_value_eq(f, x0, x)
            difference_quotient(f, x0, x) = Real.0
            compose(g, f, x) = g(f(x))
            compose(g, f, x0) = g(f(x0))
            compose(g, f, x) = compose(g, f, x0)
            difference_quotient_eq_zero_of_value_eq(compose(g, f), x0, x)
            difference_quotient(compose(g, f), x0, x) = Real.0
            chain_difference_factor(g, f(x0), dg, f(x)) * difference_quotient(f, x0, x) = dg * Real.0
            dg * Real.0 = Real.0
            chain_difference_factor(g, f(x0), dg, f(x)) * difference_quotient(f, x0, x) = Real.0
            difference_quotient(compose(g, f), x0, x) =
                chain_difference_factor(g, f(x0), dg, f(x)) * difference_quotient(f, x0, x)
        } else {
            chain_difference_factor_off_base(g, f(x0), dg, f(x))
            chain_difference_factor(g, f(x0), dg, f(x)) = difference_quotient(g, f(x0), f(x))
            difference_quotient_compose_nonzero_inner(f, g, x0, x)
            difference_quotient(g, f(x0), f(x)) * difference_quotient(f, x0, x) =
                difference_quotient(compose(g, f), x0, x)
            chain_difference_factor(g, f(x0), dg, f(x)) * difference_quotient(f, x0, x) =
                difference_quotient(compose(g, f), x0, x)
            difference_quotient(compose(g, f), x0, x) =
                chain_difference_factor(g, f(x0), dg, f(x)) * difference_quotient(f, x0, x)
        }
    }
}

/// The completed chain-rule secant factor converges to the outer derivative along the inner function.
theorem chain_difference_factor_close(
    f: Real -> Real, g: Real -> Real, x0: Real, dg: Real,
    delta_outer: Real, eps: Real, x: Real
) {
    has_derivative_at(g, f(x0), dg) and f(x).is_close(f(x0), delta_outer)
    and delta_outer.is_positive and eps.is_positive
    and forall(y: Real) {
        y != f(x0) and y.is_close(f(x0), delta_outer)
        implies difference_quotient(g, f(x0), y).is_close(dg, eps)
    }
    implies chain_difference_factor(g, f(x0), dg, f(x)).is_close(dg, eps)
} by {
    if f(x) = f(x0) {
        chain_difference_factor_at_base(g, f(x0), dg)
        chain_difference_factor(g, f(x0), dg, f(x)) = dg
        self_close(dg, eps)
        chain_difference_factor(g, f(x0), dg, f(x)).is_close(dg, eps)
    } else {
        chain_difference_factor_off_base(g, f(x0), dg, f(x))
        chain_difference_factor(g, f(x0), dg, f(x)) = difference_quotient(g, f(x0), f(x))
        f(x) != f(x0) and f(x).is_close(f(x0), delta_outer)
        derivative_close_apply(g, f(x0), dg, delta_outer, eps, f(x))
        difference_quotient(g, f(x0), f(x)).is_close(dg, eps)
        chain_difference_factor(g, f(x0), dg, f(x)).is_close(dg, eps)
    }
}

/// Bounded completed chain-rule factors and inner difference quotients give product closeness.
theorem chain_product_close(
    f: Real -> Real, g: Real -> Real, x0: Real, x: Real,
    df: Real, dg: Real, eps_small: Real, eps: Real,
    factor_bound: Real, df_bound: Real
) {
    chain_difference_factor(g, f(x0), dg, f(x)).is_close(dg, eps_small) and
    difference_quotient(f, x0, x).is_close(df, eps_small) and
    chain_difference_factor(g, f(x0), dg, f(x)).abs <= factor_bound and
    df.abs <= df_bound and factor_bound.is_positive and df_bound.is_positive and
    eps_small.is_positive and
    factor_bound * eps_small + eps_small * df_bound < eps
    implies
    (chain_difference_factor(g, f(x0), dg, f(x)) * difference_quotient(f, x0, x)).is_close(dg * df, eps)
} by {
    if chain_difference_factor(g, f(x0), dg, f(x)).is_close(dg, eps_small) and
       difference_quotient(f, x0, x).is_close(df, eps_small) and
       chain_difference_factor(g, f(x0), dg, f(x)).abs <= factor_bound and
       df.abs <= df_bound and factor_bound.is_positive and df_bound.is_positive and
       eps_small.is_positive and
       factor_bound * eps_small + eps_small * df_bound < eps {
        chain_difference_factor(g, f(x0), dg, f(x)).is_close(dg, eps_small)
        difference_quotient(f, x0, x).is_close(df, eps_small)
        chain_difference_factor(g, f(x0), dg, f(x)).abs <= factor_bound
        df.abs <= df_bound
        factor_bound.is_positive
        df_bound.is_positive
        eps_small.is_positive
        mul_close_from_close(
            chain_difference_factor(g, f(x0), dg, f(x)),
            dg,
            difference_quotient(f, x0, x),
            df,
            eps_small,
            eps_small,
            factor_bound,
            df_bound
        )
        (chain_difference_factor(g, f(x0), dg, f(x)) * difference_quotient(f, x0, x)).is_close(
            dg * df,
            factor_bound * eps_small + eps_small * df_bound
        )
        close_and_lt_imp_close(
            chain_difference_factor(g, f(x0), dg, f(x)) * difference_quotient(f, x0, x),
            dg * df,
            factor_bound * eps_small + eps_small * df_bound,
            eps
        )
        (chain_difference_factor(g, f(x0), dg, f(x)) * difference_quotient(f, x0, x)).is_close(dg * df, eps)
    }
}

/// Composition satisfies the chain rule for derivatives.
theorem derivative_compose(f: Real -> Real, g: Real -> Real, x0: Real, df: Real, dg: Real) {
    has_derivative_at(f, x0, df) and has_derivative_at(g, f(x0), dg)
    implies has_derivative_at(compose(g, f), x0, dg * df)
} by {
    derivative_continuous_at(f, x0, df)
    continuous_at(f, x0)
    continuous_at(f, x0) = forall(eps: Real) {
        eps.is_positive implies exists(delta: Real) {
            delta.is_positive and continuous_condition(f, x0, delta, eps)
        }
    }
    has_derivative_at(f, x0, df) = forall(eps: Real) {
        eps.is_positive implies exists(delta: Real) {
            delta.is_positive and forall(x: Real) {
                x != x0 and x.is_close(x0, delta)
                implies difference_quotient(f, x0, x).is_close(df, eps)
            }
        }
    }
    has_derivative_at(g, f(x0), dg) = forall(eps: Real) {
        eps.is_positive implies exists(delta: Real) {
            delta.is_positive and forall(y: Real) {
                y != f(x0) and y.is_close(f(x0), delta)
                implies difference_quotient(g, f(x0), y).is_close(dg, eps)
            }
        }
    }
    forall(eps: Real) {
        if eps.is_positive {
            let factor_bound = dg.abs + Real.1
            abs_add_one_pos(dg)
            factor_bound.is_positive
            let df_bound = df.abs + Real.1
            abs_add_one_pos(df)
            df_bound.is_positive
            let big_bound = factor_bound + df_bound
            factor_bound > Real.0
            df_bound > Real.0
            factor_bound < factor_bound + df_bound
            big_bound > Real.0
            big_bound.is_positive
            exists_small_mul_variant_2(big_bound, eps)
            let eps_small: Real satisfy {
                eps_small.is_positive and eps_small * big_bound < eps
            }
            let delta_outer: Real satisfy {
                delta_outer.is_positive and forall(y: Real) {
                    y != f(x0) and y.is_close(f(x0), delta_outer)
                    implies difference_quotient(g, f(x0), y).is_close(dg, eps_small)
                }
            }
            let delta_outer_bound: Real satisfy {
                delta_outer_bound.is_positive and forall(y: Real) {
                    y != f(x0) and y.is_close(f(x0), delta_outer_bound)
                    implies difference_quotient(g, f(x0), y).is_close(dg, Real.1)
                }
            }
            let delta_f: Real satisfy {
                delta_f.is_positive and forall(x: Real) {
                    x != x0 and x.is_close(x0, delta_f)
                    implies difference_quotient(f, x0, x).is_close(df, eps_small)
                }
            }
            let delta_cont: Real satisfy {
                delta_cont.is_positive and continuous_condition(f, x0, delta_cont, delta_outer)
            }
            let delta_cont_bound: Real satisfy {
                delta_cont_bound.is_positive and continuous_condition(f, x0, delta_cont_bound, delta_outer_bound)
            }
            eps_smaller_than_both(delta_f, delta_cont)
            let delta_12: Real satisfy {
                delta_12.is_positive and delta_12 < delta_f and delta_12 < delta_cont
            }
            eps_smaller_than_both(delta_cont_bound, delta_12)
            let delta: Real satisfy {
                delta.is_positive and delta < delta_cont_bound and delta < delta_12
            }
            forall(x: Real) {
                if x != x0 and x.is_close(x0, delta) {
                    close_and_lt_imp_close(x, x0, delta, delta_12)
                    close_and_lt_imp_close(x, x0, delta_12, delta_f)
                    close_and_lt_imp_close(x, x0, delta_12, delta_cont)
                    close_and_lt_imp_close(x, x0, delta, delta_cont_bound)
                    x.is_close(x0, delta_f)
                    x.is_close(x0, delta_cont)
                    x.is_close(x0, delta_cont_bound)
                    continuous_condition(f, x0, delta_cont, delta_outer) = forall(y: Real) {
                        y.is_close(x0, delta_cont) implies f(y).is_close(f(x0), delta_outer)
                    }
                    continuous_condition(f, x0, delta_cont_bound, delta_outer_bound) = forall(y: Real) {
                        y.is_close(x0, delta_cont_bound) implies f(y).is_close(f(x0), delta_outer_bound)
                    }
                    x != x0 and x.is_close(x0, delta_f)
                    derivative_close_apply(f, x0, df, delta_f, eps_small, x)
                    difference_quotient(f, x0, x).is_close(df, eps_small)
                    f(x).is_close(f(x0), delta_outer)
                    f(x).is_close(f(x0), delta_outer_bound)
                    has_derivative_at(g, f(x0), dg)
                    chain_difference_factor_close(f, g, x0, dg, delta_outer, eps_small, x)
                    chain_difference_factor(g, f(x0), dg, f(x)).is_close(dg, eps_small)
                    Real.1.is_positive
                    chain_difference_factor_close(f, g, x0, dg, delta_outer_bound, Real.1, x)
                    chain_difference_factor(g, f(x0), dg, f(x)).is_close(dg, Real.1)
                    abs_le_abs_add_eps(chain_difference_factor(g, f(x0), dg, f(x)), dg, Real.1)
                    chain_difference_factor(g, f(x0), dg, f(x)).abs <= dg.abs + Real.1
                    chain_difference_factor(g, f(x0), dg, f(x)).abs <= factor_bound
                    abs_lte_abs_add_one(df)
                    df.abs <= df_bound
                    let total = factor_bound * eps_small + eps_small * df_bound
                    factor_bound * eps_small = eps_small * factor_bound
                    total = eps_small * factor_bound + eps_small * df_bound
                    eps_small * factor_bound + eps_small * df_bound = eps_small * (factor_bound + df_bound)
                    total = eps_small * big_bound
                    total < eps
                    chain_product_close(f, g, x0, x, df, dg, eps_small, eps, factor_bound, df_bound)
                    (chain_difference_factor(g, f(x0), dg, f(x)) * difference_quotient(f, x0, x)).is_close(dg * df, eps)
                    difference_quotient_compose_factor(f, g, x0, x, dg)
                    difference_quotient(compose(g, f), x0, x) =
                        chain_difference_factor(g, f(x0), dg, f(x)) * difference_quotient(f, x0, x)
                    close_eq_left(
                        chain_difference_factor(g, f(x0), dg, f(x)) * difference_quotient(f, x0, x),
                        difference_quotient(compose(g, f), x0, x),
                        dg * df,
                        eps
                    )
                    difference_quotient(compose(g, f), x0, x).is_close(dg * df, eps)
                }
            }
            delta.is_positive and forall(x: Real) {
                x != x0 and x.is_close(x0, delta)
                implies difference_quotient(compose(g, f), x0, x).is_close(dg * df, eps)
            }
            exists(delta2: Real) {
                delta2.is_positive and forall(x: Real) {
                    x != x0 and x.is_close(x0, delta2)
                    implies difference_quotient(compose(g, f), x0, x).is_close(dg * df, eps)
                }
            }
        }
    }
    has_derivative_at(compose(g, f), x0, dg * df) = forall(eps: Real) {
        eps.is_positive implies exists(delta: Real) {
            delta.is_positive and forall(x: Real) {
                x != x0 and x.is_close(x0, delta)
                implies difference_quotient(compose(g, f), x0, x).is_close(dg * df, eps)
            }
        }
    }
    if not has_derivative_at(compose(g, f), x0, dg * df) {
        not forall(eps: Real) {
            eps.is_positive implies exists(delta: Real) {
                delta.is_positive and forall(x: Real) {
                    x != x0 and x.is_close(x0, delta)
                    implies difference_quotient(compose(g, f), x0, x).is_close(dg * df, eps)
                }
            }
        }
        let bad_eps: Real satisfy {
            bad_eps.is_positive and forall(delta: Real) {
                not (delta.is_positive and forall(x: Real) {
                    x != x0 and x.is_close(x0, delta)
                    implies difference_quotient(compose(g, f), x0, x).is_close(dg * df, bad_eps)
                })
            }
        }
        exists(delta: Real) {
            delta.is_positive and forall(x: Real) {
                x != x0 and x.is_close(x0, delta)
                implies difference_quotient(compose(g, f), x0, x).is_close(dg * df, bad_eps)
            }
        }
        false
    }
}

/// Differentiability is preserved by composition when the outer function is differentiable at the inner value.
theorem differentiable_compose(f: Real -> Real, g: Real -> Real, x0: Real) {
    differentiable_at(f, x0) and differentiable_at(g, f(x0))
    implies differentiable_at(compose(g, f), x0)
} by {
    let df: Real satisfy {
        has_derivative_at(f, x0, df)
    }
    let dg: Real satisfy {
        has_derivative_at(g, f(x0), dg)
    }
    derivative_compose(f, g, x0, df, dg)
    has_derivative_at(compose(g, f), x0, dg * df)
    exists(d: Real) {
        has_derivative_at(compose(g, f), x0, d)
    }
}
