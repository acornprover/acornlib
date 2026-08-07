from data.basic.set import Set, compl_contains_eq, double_inclusion, empty_set_contains_eq,
    intersection_contains_intro, set_eq_empty_of_is_empty
from real.real_field import Real
from real.topology import closure, interior, is_closed_set, is_open_set, point_in_closure_if_in_set,
    set_subset_closure
from real.topology_boundary import boundary, boundary_complement_eq,
    boundary_contains_eq, boundary_subset_complement_of_open_set, boundary_subset_of_closed_set
from real.topology_clopen import clopen_real_set_eq_closure, clopen_real_set_eq_interior,
    clopen_real_set_intro, is_clopen_real_set
from real.topology_dense import is_dense_real_set
from real.topology_interior_closure_idempotent import closed_set_iff_eq_closure,
    open_set_iff_complement_closed

/// A clopen real set has empty boundary.
theorem boundary_of_clopen_real_set_is_empty(s: Set[Real]) {
    is_clopen_real_set(s) implies boundary(s) = Set[Real].empty_set
} by {
    if is_clopen_real_set(s) {
        boundary_subset_of_closed_set(s)
        boundary_subset_complement_of_open_set(s)
        boundary(s).subset(s)
        boundary(s).subset(s.c)
        forall(x: Real) {
            if boundary(s).contains(x) {
                s.contains(x)
                s.c.contains(x)
                compl_contains_eq(s, x)
                not s.contains(x)
                false
            }
        }
        boundary(s).is_empty
        set_eq_empty_of_is_empty[Real](boundary(s))
        boundary(s) = Set[Real].empty_set
    }
}

/// If a real set has empty boundary, it is closed.
theorem closed_real_set_of_empty_boundary(s: Set[Real]) {
    boundary(s) = Set[Real].empty_set implies is_closed_set(s)
} by {
    if boundary(s) = Set[Real].empty_set {
        forall(x: Real) {
            if closure(s).contains(x) {
                if not s.contains(x) {
                    compl_contains_eq(s, x)
                    s.c.contains(x)
                    point_in_closure_if_in_set(s.c, x)
                    closure(s.c).contains(x)
                    intersection_contains_intro(closure(s), closure(s.c), x)
                    boundary_contains_eq(s, x)
                    boundary(s).contains(x)
                    Set[Real].empty_set.contains(x)
                    empty_set_contains_eq[Real](x)
                    false
                }
                s.contains(x)
            }
        }
        closure(s).subset(s)
        set_subset_closure(s)
        s.subset(closure(s))
        double_inclusion(s, closure(s))
        s = closure(s)
        closed_set_iff_eq_closure(s)
        is_closed_set(s)
    }
}

/// If a real set has empty boundary, it is open.
theorem open_real_set_of_empty_boundary(s: Set[Real]) {
    boundary(s) = Set[Real].empty_set implies is_open_set(s)
} by {
    if boundary(s) = Set[Real].empty_set {
        boundary_complement_eq(s)
        boundary(s.c) = boundary(s)
        boundary(s.c) = Set[Real].empty_set
        closed_real_set_of_empty_boundary(s.c)
        is_closed_set(s.c)
        open_set_iff_complement_closed(s)
        is_open_set(s)
    }
}

/// If a real set has empty boundary, it is clopen.
theorem clopen_real_set_of_empty_boundary(s: Set[Real]) {
    boundary(s) = Set[Real].empty_set implies is_clopen_real_set(s)
} by {
    if boundary(s) = Set[Real].empty_set {
        open_real_set_of_empty_boundary(s)
        closed_real_set_of_empty_boundary(s)
        is_open_set(s)
        is_closed_set(s)
        clopen_real_set_intro(s)
        is_clopen_real_set(s)
    }
}

/// A real set has empty boundary exactly when it is clopen.
theorem boundary_empty_eq_clopen_real_set(s: Set[Real]) {
    (boundary(s) = Set[Real].empty_set) = is_clopen_real_set(s)
} by {
    if boundary(s) = Set[Real].empty_set {
        clopen_real_set_of_empty_boundary(s)
        is_clopen_real_set(s)
    }
    if is_clopen_real_set(s) {
        boundary_of_clopen_real_set_is_empty(s)
        boundary(s) = Set[Real].empty_set
    }
    (boundary(s) = Set[Real].empty_set) = is_clopen_real_set(s)
}

/// A real set with empty boundary has complement with empty boundary.
theorem boundary_complement_empty_of_empty_boundary(s: Set[Real]) {
    boundary(s) = Set[Real].empty_set implies boundary(s.c) = Set[Real].empty_set
} by {
    if boundary(s) = Set[Real].empty_set {
        boundary_complement_eq(s)
        boundary(s.c) = boundary(s)
        boundary(s.c) = Set[Real].empty_set
    }
}

/// The complement of a set with empty boundary is clopen.
theorem complement_clopen_of_empty_boundary(s: Set[Real]) {
    boundary(s) = Set[Real].empty_set implies is_clopen_real_set(s.c)
} by {
    if boundary(s) = Set[Real].empty_set {
        boundary_complement_empty_of_empty_boundary(s)
        boundary(s.c) = Set[Real].empty_set
        clopen_real_set_of_empty_boundary(s.c)
        is_clopen_real_set(s.c)
    }
}

/// Empty boundary is invariant under complement.
theorem boundary_complement_empty_eq_boundary_empty(s: Set[Real]) {
    (boundary(s.c) = Set[Real].empty_set) = (boundary(s) = Set[Real].empty_set)
} by {
    if boundary(s.c) = Set[Real].empty_set {
        boundary_complement_eq(s)
        boundary(s.c) = boundary(s)
        boundary(s) = Set[Real].empty_set
    }
    if boundary(s) = Set[Real].empty_set {
        boundary_complement_empty_of_empty_boundary(s)
        boundary(s.c) = Set[Real].empty_set
    }
    (boundary(s.c) = Set[Real].empty_set) = (boundary(s) = Set[Real].empty_set)
}

/// A set with empty boundary has equal closure and interior.
theorem closure_eq_interior_of_empty_boundary(s: Set[Real]) {
    boundary(s) = Set[Real].empty_set implies closure(s) = interior(s)
} by {
    if boundary(s) = Set[Real].empty_set {
        clopen_real_set_of_empty_boundary(s)
        is_clopen_real_set(s)
        clopen_real_set_eq_closure(s)
        clopen_real_set_eq_interior(s)
        s = closure(s)
        s = interior(s)
        closure(s) = interior(s)
    }
}

/// A dense clopen real set is universal.
theorem dense_clopen_real_set_is_universal(s: Set[Real]) {
    is_dense_real_set(s) and is_clopen_real_set(s) implies s = Set[Real].universal_set
} by {
    if is_dense_real_set(s) and is_clopen_real_set(s) {
        is_dense_real_set(s) = (closure(s) = Set[Real].universal_set)
        closure(s) = Set[Real].universal_set
        clopen_real_set_eq_closure(s)
        s = closure(s)
        s = Set[Real].universal_set
    }
}

/// A dense real set with empty boundary is universal.
theorem dense_empty_boundary_real_set_is_universal(s: Set[Real]) {
    is_dense_real_set(s) and boundary(s) = Set[Real].empty_set implies s = Set[Real].universal_set
} by {
    if is_dense_real_set(s) and boundary(s) = Set[Real].empty_set {
        clopen_real_set_of_empty_boundary(s)
        is_clopen_real_set(s)
        dense_clopen_real_set_is_universal(s)
        s = Set[Real].universal_set
    }
}

/// A closed dense real set with empty-boundary complement is universal.
theorem closed_dense_real_set_from_empty_boundary_complement_is_universal(s: Set[Real]) {
    is_dense_real_set(s) and boundary(s.c) = Set[Real].empty_set implies s = Set[Real].universal_set
} by {
    if is_dense_real_set(s) and boundary(s.c) = Set[Real].empty_set {
        boundary_complement_empty_eq_boundary_empty(s)
        boundary(s) = Set[Real].empty_set
        dense_empty_boundary_real_set_is_universal(s)
        s = Set[Real].universal_set
    }
}
