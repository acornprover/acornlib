/// The infrastructure for the one-dimensional implicit function theorem.
///
/// A curried two-variable function f: Real -> Real -> Real is given,
/// together with its partial derivatives fx(x, y) = df/dx (the derivative
/// of the slice x -> f(x, y)) and fy(x, y) = df/dy (the derivative of the
/// slice y -> f(x, y)).  This file provides the machinery needed to prove
/// the derivative formula
///
///     g'(x0) = -fx(x0, y0) / fy(x0, y0)
///
/// for a function g defined implicitly by f(x, g(x)) = 0 with g(x0) = y0:
///
///   * `partial_x_fn` and `partial_y_fn`: the slices of f at a fixed second
///     or first argument;
///   * `continuous_2`: two-variable continuity of a curried function at a
///     point, with the `continuous_2_apply` application lemma;
///   * `mvt_x_slice`, `mvt_x_slice_gt`, `mvt_y_slice`, `mvt_y_slice_gt`:
///     the mean value theorem applied to the slices, in both endpoint
///     orientations, giving the identities
///     f(b, y) - f(a, y) = fx(c, y) * (b - a) and
///     f(x, b) - f(x, a) = fy(x, c) * (b - a);
///   * `between_imp_closer_right` and `between_imp_closer_left`: a point
///     strictly between x0 and q is closer to x0 than q is;
///   * `ratio_close`: the quotient-continuity bound
///     |a/b - a0/b0| < eps under closeness of a to a0 and b to a nonzero
///     b0 with a suitable tolerance budget.
///
/// The full statement of the theorem (that g is differentiable at x0 with
/// the derivative above) is documented in a comment at the end of the file;
/// its ε-δ proof is future work.  The existence part (that such a g exists)
/// is also not proved here; it requires the intermediate value theorem on
/// the slices and monotonicity from fy != 0.

from nat import Nat
from real.real_field import Real
from real.derivative_basic import has_derivative_at, has_derivative_at_unique, difference_quotient, sub_ne_zero_of_ne
from real.calculus_api import is_derivative_fn, is_derivative_fn_at
from real.continuity_base import continuous, continuous_at, continuous_condition
from real.continuity_sequences import continuous_at_preserves_sequence_limit
from real.mean_value import mean_value_theorem, secant_slope, lte_imp_neg_lte_neg
from real.real_seq import converges_to, tail_bound, tail_bound_implies_is_close, lt_imp_minus_pos, eps_smaller_than_both, sub_lt_is_gt, close_and_lt_imp_close
from real.real_base import close_imp_bounds, bounds_imp_close, abs_gte_zero, lte_lt_trans, lt_lte_trans, lt_add_pos, add_lt_lt, pos_imp_eq_abs, gt_zero_imp_pos, pos_gt_zero, lt_add_right, abs_neg, add_zero_right, add_zero_left, one_half_positive, one_half_plus_one_half, add_comm, add_assoc, add_neg_eq_zero, neg_distrib, neg_neg
from real.real_series import triangle_ineq
from real.exp import two, two_positive, two_nonzero, abs_div, div_lt_div_pos
from real.uniform_continuity import div_le_div_pos
from real.real_ring import mul_abs, mul_pos_pos, mul_one_right, mul_one_left, real_mul_comm, mul_neg_left, mul_neg_right, mul_distrib_left, mul_sub_distrib_left, mul_sub_distrib_right, mul_assoc, mul_nonneg
from real.real_field import div_lt_mul_pos, mul_div_cancel, mul_inverse, div_mul_cancel_left, mul_left_cancel, mul_le_mul_pos_right, div_cancel_common
from real.derivative_continuity import div_mul_cancel_denominator
from real.am_gm import div_pos_of_pos_pos, div_nonneg_of_nonneg_pos
from real.trig_identities import div_sub_same_denom
from algebra.field.field import inverse_not_zero, mul_not_zero, inverse_dist
from ordered_field import mul_lt_mul_of_pos_right, multiply_inequality_with_nonnegative_element, inverse_on_positive_flips_inequality
from algebra.add_ordered_group import add_le_add_right
from data.basic.functions import compose
from order import lt_trans, lt_imp_lte, lte_trans, lt_imp_ne, lt_of_lte_of_ne
from real.lhopital_infinity import add_nonneg_pos_pos, add_pos_pos, lt_mul_pos_imp_div_lt, add_ge_imp_ge_sub
from real.lhopital import neg_sub_distrib

numerals Real

// =============================================================================
// Curried partial derivatives and two-variable continuity
// =============================================================================

/// The slice x -> f(x, y) of a curried two-variable function at fixed second
/// argument y.
define partial_x_fn(f: Real -> Real -> Real, y: Real, x: Real) -> Real {
    f(x, y)
}

/// The slice y -> f(x, y) of a curried two-variable function at fixed first
/// argument x.
define partial_y_fn(f: Real -> Real -> Real, x: Real, y: Real) -> Real {
    f(x, y)
}

/// True if the curried two-variable function f is continuous at (x0, y0),
/// in ε-δ form: near (x0, y0) the value f(x, y) stays within eps of
/// f(x0, y0).
define continuous_2(f: Real -> Real -> Real, x0: Real, y0: Real) -> Bool {
    forall(eps: Real) {
        eps.is_positive implies exists(delta: Real) {
            delta.is_positive and forall(x: Real, y: Real) {
                x.is_close(x0, delta) and y.is_close(y0, delta)
                implies f(x, y).is_close(f(x0, y0), eps)
            }
        }
    }
}

/// Applying the two-variable continuity hypothesis at a tolerance yields a
/// delta.
theorem continuous_2_apply(f: Real -> Real -> Real, x0: Real, y0: Real, eps: Real) {
    continuous_2(f, x0, y0) and eps.is_positive
    implies exists(delta: Real) {
        delta.is_positive and forall(x: Real, y: Real) {
            x.is_close(x0, delta) and y.is_close(y0, delta)
            implies f(x, y).is_close(f(x0, y0), eps)
        }
    }
} by {
    if continuous_2(f, x0, y0) and eps.is_positive {
        continuous_2(f, x0, y0) = forall(eps2: Real) {
            eps2.is_positive implies exists(delta2: Real) {
                delta2.is_positive and forall(x: Real, y: Real) {
                    x.is_close(x0, delta2) and y.is_close(y0, delta2)
                    implies f(x, y).is_close(f(x0, y0), eps2)
                }
            }
        }
        exists(delta: Real) {
            delta.is_positive and forall(x: Real, y: Real) {
                x.is_close(x0, delta) and y.is_close(y0, delta)
                implies f(x, y).is_close(f(x0, y0), eps)
            }
        }
    }
}

// =============================================================================
// The mean value theorem on slices
// =============================================================================

/// The mean value theorem on the x-slice: f(b, y) - f(a, y) = fx(c, y) *
/// (b - a) for some c between a and b.
theorem mvt_x_slice(f: Real -> Real -> Real, fx: Real -> Real -> Real,
    a: Real, b: Real, y: Real) {
    continuous(partial_x_fn(f, y)) and
    is_derivative_fn(partial_x_fn(f, y), partial_x_fn(fx, y)) and a < b
    implies exists(c: Real) {
        a < c and c < b and f(b, y) - f(a, y) = fx(c, y) * (b - a)
    }
} by {
    if continuous(partial_x_fn(f, y)) and
       is_derivative_fn(partial_x_fn(f, y), partial_x_fn(fx, y)) and a < b {
        mean_value_theorem(partial_x_fn(f, y), partial_x_fn(fx, y), a, b)
        let c: Real satisfy {
            a < c and c < b and has_derivative_at(partial_x_fn(f, y), c,
                secant_slope(partial_x_fn(f, y), a, b))
        }
        a < c and c < b
        has_derivative_at(partial_x_fn(f, y), c,
            secant_slope(partial_x_fn(f, y), a, b))
        is_derivative_fn_at(partial_x_fn(f, y), partial_x_fn(fx, y), c)
        has_derivative_at(partial_x_fn(f, y), c, partial_x_fn(fx, y, c))
        has_derivative_at_unique(partial_x_fn(f, y), c,
            secant_slope(partial_x_fn(f, y), a, b), partial_x_fn(fx, y, c))
        secant_slope(partial_x_fn(f, y), a, b) = partial_x_fn(fx, y, c)
        partial_x_fn(fx, y, c) = fx(c, y)
        secant_slope(partial_x_fn(f, y), a, b) = fx(c, y)
        secant_slope(partial_x_fn(f, y), a, b) =
            (partial_x_fn(f, y, b) - partial_x_fn(f, y, a)) / (b - a)
        partial_x_fn(f, y, b) = f(b, y)
        partial_x_fn(f, y, a) = f(a, y)
        secant_slope(partial_x_fn(f, y), a, b) = (f(b, y) - f(a, y)) / (b - a)
        (f(b, y) - f(a, y)) / (b - a) = fx(c, y)
        ((f(b, y) - f(a, y)) / (b - a)) * (b - a) = fx(c, y) * (b - a)
        lt_imp_ne(a, b)
        a != b
        sub_ne_zero_of_ne(b, a)
        b - a != Real.0
        div_mul_cancel_denominator(f(b, y) - f(a, y), b - a)
        ((f(b, y) - f(a, y)) / (b - a)) * (b - a) = f(b, y) - f(a, y)
        f(b, y) - f(a, y) = fx(c, y) * (b - a)
        exists(c2: Real) {
            a < c2 and c2 < b and f(b, y) - f(a, y) = fx(c2, y) * (b - a)
        }
    }
}

/// The mean value theorem on the x-slice with reversed endpoints:
/// F(b, y) - F(a, y) = fx(c, y) * (b - a) for some c between b and a.
theorem mvt_x_slice_gt(f: Real -> Real -> Real, fx: Real -> Real -> Real,
    a: Real, b: Real, y: Real) {
    continuous(partial_x_fn(f, y)) and
    is_derivative_fn(partial_x_fn(f, y), partial_x_fn(fx, y)) and b < a
    implies exists(c: Real) {
        b < c and c < a and f(b, y) - f(a, y) = fx(c, y) * (b - a)
    }
} by {
    if continuous(partial_x_fn(f, y)) and
       is_derivative_fn(partial_x_fn(f, y), partial_x_fn(fx, y)) and b < a {
        mvt_x_slice(f, fx, b, a, y)
        let c: Real satisfy {
            b < c and c < a and f(a, y) - f(b, y) = fx(c, y) * (a - b)
        }
        b < c and c < a
        f(a, y) - f(b, y) = fx(c, y) * (a - b)
        f(b, y) - f(a, y) = -(f(a, y) - f(b, y))
        f(b, y) - f(a, y) = -(fx(c, y) * (a - b))
        fx(c, y) * (b - a) = -(fx(c, y) * (a - b))
        f(b, y) - f(a, y) = fx(c, y) * (b - a)
        exists(c2: Real) {
            b < c2 and c2 < a and f(b, y) - f(a, y) = fx(c2, y) * (b - a)
        }
    }
}

/// The mean value theorem on the y-slice: f(x, b) - f(x, a) = fy(x, c) *
/// (b - a) for some c between a and b.
theorem mvt_y_slice(f: Real -> Real -> Real, fy: Real -> Real -> Real,
    x: Real, a: Real, b: Real) {
    continuous(partial_y_fn(f, x)) and
    is_derivative_fn(partial_y_fn(f, x), partial_y_fn(fy, x)) and a < b
    implies exists(c: Real) {
        a < c and c < b and f(x, b) - f(x, a) = fy(x, c) * (b - a)
    }
} by {
    if continuous(partial_y_fn(f, x)) and
       is_derivative_fn(partial_y_fn(f, x), partial_y_fn(fy, x)) and a < b {
        mean_value_theorem(partial_y_fn(f, x), partial_y_fn(fy, x), a, b)
        let c: Real satisfy {
            a < c and c < b and has_derivative_at(partial_y_fn(f, x), c,
                secant_slope(partial_y_fn(f, x), a, b))
        }
        a < c and c < b
        has_derivative_at(partial_y_fn(f, x), c,
            secant_slope(partial_y_fn(f, x), a, b))
        is_derivative_fn_at(partial_y_fn(f, x), partial_y_fn(fy, x), c)
        has_derivative_at(partial_y_fn(f, x), c, partial_y_fn(fy, x, c))
        has_derivative_at_unique(partial_y_fn(f, x), c,
            secant_slope(partial_y_fn(f, x), a, b), partial_y_fn(fy, x, c))
        secant_slope(partial_y_fn(f, x), a, b) = partial_y_fn(fy, x, c)
        partial_y_fn(fy, x, c) = fy(x, c)
        secant_slope(partial_y_fn(f, x), a, b) = fy(x, c)
        secant_slope(partial_y_fn(f, x), a, b) =
            (partial_y_fn(f, x, b) - partial_y_fn(f, x, a)) / (b - a)
        partial_y_fn(f, x, b) = f(x, b)
        partial_y_fn(f, x, a) = f(x, a)
        secant_slope(partial_y_fn(f, x), a, b) = (f(x, b) - f(x, a)) / (b - a)
        (f(x, b) - f(x, a)) / (b - a) = fy(x, c)
        ((f(x, b) - f(x, a)) / (b - a)) * (b - a) = fy(x, c) * (b - a)
        lt_imp_ne(a, b)
        a != b
        sub_ne_zero_of_ne(b, a)
        b - a != Real.0
        div_mul_cancel_denominator(f(x, b) - f(x, a), b - a)
        ((f(x, b) - f(x, a)) / (b - a)) * (b - a) = f(x, b) - f(x, a)
        f(x, b) - f(x, a) = fy(x, c) * (b - a)
        exists(c2: Real) {
            a < c2 and c2 < b and f(x, b) - f(x, a) = fy(x, c2) * (b - a)
        }
    }
}

/// The mean value theorem on the y-slice with reversed endpoints:
/// f(x, b) - f(x, a) = fy(x, c) * (b - a) for some c between b and a.
theorem mvt_y_slice_gt(f: Real -> Real -> Real, fy: Real -> Real -> Real,
    x: Real, a: Real, b: Real) {
    continuous(partial_y_fn(f, x)) and
    is_derivative_fn(partial_y_fn(f, x), partial_y_fn(fy, x)) and b < a
    implies exists(c: Real) {
        b < c and c < a and f(x, b) - f(x, a) = fy(x, c) * (b - a)
    }
} by {
    if continuous(partial_y_fn(f, x)) and
       is_derivative_fn(partial_y_fn(f, x), partial_y_fn(fy, x)) and b < a {
        mvt_y_slice(f, fy, x, b, a)
        let c: Real satisfy {
            b < c and c < a and f(x, a) - f(x, b) = fy(x, c) * (a - b)
        }
        b < c and c < a
        f(x, a) - f(x, b) = fy(x, c) * (a - b)
        f(x, b) - f(x, a) = -(f(x, a) - f(x, b))
        f(x, b) - f(x, a) = -(fy(x, c) * (a - b))
        fy(x, c) * (b - a) = -(fy(x, c) * (a - b))
        f(x, b) - f(x, a) = fy(x, c) * (b - a)
        exists(c2: Real) {
            b < c2 and c2 < a and f(x, b) - f(x, a) = fy(x, c2) * (b - a)
        }
    }
}

/// A point strictly between x0 and q is closer to x0 than q is, when q lies
/// to the right of x0.
theorem between_imp_closer_right(x0: Real, q: Real, c: Real) {
    (x0 < c and c < q)
    implies (c - x0).abs < (q - x0).abs
} by {
    if x0 < c and c < q {
        lt_trans[Real](x0, c, q)
        x0 < q
        lt_imp_minus_pos(x0, c)
        (c - x0).is_positive
        pos_imp_eq_abs(c - x0)
        c - x0 = (c - x0).abs
        (c - x0).abs = c - x0
        lt_add_right(c, q, -x0)
        -x0 + c < -x0 + q
        -x0 + c = c - x0
        -x0 + q = q - x0
        c - x0 < q - x0
        lt_imp_minus_pos(c, q)
        (q - c).is_positive
        pos_imp_eq_abs(q - c)
        q - c = (q - c).abs
        (q - c).abs = q - c
        lt_imp_minus_pos(x0, q)
        (q - x0).is_positive
        pos_imp_eq_abs(q - x0)
        q - x0 = (q - x0).abs
        (q - x0).abs = q - x0
        c - x0 < q - x0
        (c - x0).abs < (q - x0).abs
    }
}

/// A point strictly between x0 and q is closer to x0 than q is, when q lies
/// to the left of x0.
theorem between_imp_closer_left(x0: Real, q: Real, c: Real) {
    (q < c and c < x0)
    implies (c - x0).abs < (q - x0).abs
} by {
    if q < c and c < x0 {
        lt_trans[Real](q, c, x0)
        q < x0
        lt_imp_minus_pos(c, x0)
        (x0 - c).is_positive
        pos_imp_eq_abs(x0 - c)
        x0 - c = (x0 - c).abs
        (x0 - c).abs = x0 - c
        (c - x0).abs = (x0 - c).abs
        (c - x0).abs = x0 - c
        lt_imp_minus_pos(q, x0)
        (x0 - q).is_positive
        pos_imp_eq_abs(x0 - q)
        x0 - q = (x0 - q).abs
        (x0 - q).abs = x0 - q
        (q - x0).abs = (x0 - q).abs
        (q - x0).abs = x0 - q
        sub_lt_is_gt(x0, q, c)
        x0 - q > x0 - c
        x0 - c < x0 - q
        (c - x0).abs < (q - x0).abs
    }
}



// =============================================================================
// The derivative of the implicitly defined function
// =============================================================================

/// The ratio (a / b) - (a0 / b0) is small when a is close to a0, b is close
/// to a nonzero b0, and the tolerances fit the budget
/// 2 * (eps_a * |b0| + |a0| * eps_b) < eps * |b0|^2.
theorem ratio_close(a: Real, a0: Real, b: Real, b0: Real,
    eps_a: Real, eps_b: Real, eps: Real) {
    (a - a0).abs < eps_a and (b - b0).abs < eps_b and
    b != Real.0 and b0 != Real.0 and
    eps_b <= b0.abs * Real.one_half and
    (two * (eps_a * b0.abs + a0.abs * eps_b) < eps * (b0.abs * b0.abs))
    implies (a / b - a0 / b0).abs < eps
} by {
    if (a - a0).abs < eps_a and (b - b0).abs < eps_b and
       b != Real.0 and b0 != Real.0 and
       eps_b <= b0.abs * Real.one_half and
       (two * (eps_a * b0.abs + a0.abs * eps_b) < eps * (b0.abs * b0.abs)) {
        // |b0| is positive.
        abs_gte_zero(b0)
        b0.abs >= Real.0
        b0.abs != Real.0
        lt_imp_lte(Real.0, b0.abs)
        Real.0 <= b0.abs
        lt_imp_ne(Real.0, b0.abs)
        Real.0 != b0.abs
        lt_of_lte_of_ne[Real](Real.0, b0.abs)
        Real.0 < b0.abs
        gt_zero_imp_pos(b0.abs)
        b0.abs.is_positive
        // |b| > |b0| - eps_b >= |b0|/2, so |b| > 0.
        triangle_ineq(b, -(b - b0))
        (b + -(b - b0)).abs <= b.abs + (-(b - b0)).abs
        neg_sub_distrib(b, b0)
        -(b - b0) = -b + b0
        b + -(b - b0) = b + (-b + b0)
        add_assoc(b, -b, b0)
        (b + -b) + b0 = b + (-b + b0)
        add_neg_eq_zero(b)
        b + -b = Real.0
        (b + -b) + b0 = Real.0 + b0
        add_zero_left(b0)
        Real.0 + b0 = b0
        b + -(b - b0) = b0
        b0.abs <= b.abs + (-(b - b0)).abs
        abs_neg(b - b0)
        (-(b - b0)).abs = (b - b0).abs
        b0.abs <= b.abs + (b - b0).abs
        add_ge_imp_ge_sub(b.abs, (b - b0).abs, b0.abs)
        b.abs >= b0.abs - (b - b0).abs
        b0.abs - (b - b0).abs <= b.abs
        sub_lt_is_gt(b0.abs, (b - b0).abs, eps_b)
        b0.abs - (b - b0).abs > b0.abs - eps_b
        lt_lte_trans(b0.abs - eps_b, b0.abs - (b - b0).abs, b.abs)
        b0.abs - eps_b < b.abs
        one_half_positive
        Real.0 < Real.one_half
        gt_zero_imp_pos(Real.one_half)
        Real.one_half.is_positive
        mul_pos_pos(b0.abs, Real.one_half)
        (b0.abs * Real.one_half).is_positive
        gt_zero_imp_pos(b0.abs * Real.one_half)
        b0.abs * Real.one_half > Real.0
        lte_imp_neg_lte_neg(eps_b, b0.abs * Real.one_half)
        -(b0.abs * Real.one_half) <= -eps_b
        add_le_add_right(-(b0.abs * Real.one_half), -eps_b, b0.abs)
        b0.abs + -(b0.abs * Real.one_half) <= b0.abs + -eps_b
        b0.abs - b0.abs * Real.one_half = b0.abs * Real.one_half
        b0.abs * Real.one_half <= b0.abs - eps_b
        lte_lt_trans(b0.abs * Real.one_half, b0.abs - eps_b, b.abs)
        b0.abs * Real.one_half < b.abs
        gt_zero_imp_pos(b.abs)
        b.abs.is_positive
        pos_gt_zero(b.abs)
        b.abs > Real.0
        // |b| * |b0| > (|b0|/2) * |b0|.
        mul_lt_mul_of_pos_right[Real](b0.abs * Real.one_half, b.abs, b0.abs)
        (b0.abs * Real.one_half) * b0.abs < b.abs * b0.abs
        real_mul_comm(b0.abs * Real.one_half, b0.abs)
        (b0.abs * Real.one_half) * b0.abs = b0.abs * (b0.abs * Real.one_half)
        b0.abs * (b0.abs * Real.one_half) < b.abs * b0.abs
        // |b| * |b0| > 0.
        mul_pos_pos(b.abs, b0.abs)
        (b.abs * b0.abs).is_positive
        gt_zero_imp_pos(b.abs * b0.abs)
        b.abs * b0.abs > Real.0
        // The numerator identity: a*b0 - a0*b = (a-a0)*b0 + a0*(b0-b).
        mul_sub_distrib_left(a, a0, b0)
        (a - a0) * b0 = a * b0 - a0 * b0
        mul_sub_distrib_right(a0, b0, b)
        a0 * (b0 - b) = a0 * b0 - a0 * b
        (a - a0) * b0 + a0 * (b0 - b) = (a * b0 - a0 * b0) + (a0 * b0 - a0 * b)
        (a * b0 - a0 * b0) + (a0 * b0 - a0 * b) = a * b0 - a0 * b
        // |a*b0 - a0*b| <= |a-a0|*|b0| + |a0|*|b-b0| <= S.
        triangle_ineq((a - a0) * b0, a0 * (b0 - b))
        ((a - a0) * b0 + a0 * (b0 - b)).abs <= ((a - a0) * b0).abs + (a0 * (b0 - b)).abs
        mul_abs(a - a0, b0)
        (a - a0).abs * b0.abs = ((a - a0) * b0).abs
        ((a - a0) * b0).abs = (a - a0).abs * b0.abs
        mul_abs(a0, b0 - b)
        a0.abs * (b0 - b).abs = (a0 * (b0 - b)).abs
        (a0 * (b0 - b)).abs = a0.abs * (b0 - b).abs
        ((a - a0) * b0 + a0 * (b0 - b)).abs <= (a - a0).abs * b0.abs + a0.abs * (b0 - b).abs
        multiply_inequality_with_nonnegative_element[Real](
            (a - a0).abs, eps_a, b0.abs)
        (a - a0).abs * b0.abs <= eps_a * b0.abs
        abs_gte_zero(a0)
        a0.abs >= Real.0
        abs_gte_zero(b - b0)
        (b - b0).abs >= Real.0
        (b - b0).abs < eps_b
        lte_lt_trans(Real.0, (b - b0).abs, eps_b)
        Real.0 < eps_b
        lt_imp_lte((b - b0).abs, eps_b)
        (b - b0).abs <= eps_b
        abs_gte_zero(a0)
        a0.abs >= Real.0
        multiply_inequality_with_nonnegative_element[Real](
            (b - b0).abs, eps_b, a0.abs)
        (b - b0).abs * a0.abs <= eps_b * a0.abs
        real_mul_comm((b - b0).abs, a0.abs)
        (b - b0).abs * a0.abs = a0.abs * (b - b0).abs
        a0.abs * (b - b0).abs <= eps_b * a0.abs
        abs_neg(b0 - b)
        (-(b0 - b)).abs = (b0 - b).abs
        (b0 - b).abs = (b - b0).abs
        a0.abs * (b0 - b).abs <= eps_b * a0.abs
        eps_b * a0.abs = a0.abs * eps_b
        a0.abs * (b0 - b).abs <= a0.abs * eps_b
        add_le_add_right((a - a0).abs * b0.abs, eps_a * b0.abs,
            a0.abs * (b0 - b).abs)
        (a - a0).abs * b0.abs + a0.abs * (b0 - b).abs <= eps_a * b0.abs + a0.abs * (b0 - b).abs
        add_le_add_right(a0.abs * (b0 - b).abs, a0.abs * eps_b,
            eps_a * b0.abs)
        eps_a * b0.abs + a0.abs * (b0 - b).abs <= eps_a * b0.abs + a0.abs * eps_b
        lte_trans((a - a0).abs * b0.abs + a0.abs * (b0 - b).abs,
            eps_a * b0.abs + a0.abs * (b0 - b).abs,
            eps_a * b0.abs + a0.abs * eps_b)
        (a - a0).abs * b0.abs + a0.abs * (b0 - b).abs <= eps_a * b0.abs + a0.abs * eps_b
        lte_trans(((a - a0) * b0 + a0 * (b0 - b)).abs,
            (a - a0).abs * b0.abs + a0.abs * (b0 - b).abs,
            eps_a * b0.abs + a0.abs * eps_b)
        ((a - a0) * b0 + a0 * (b0 - b)).abs <= eps_a * b0.abs + a0.abs * eps_b
        (a * b0 - a0 * b).abs <= eps_a * b0.abs + a0.abs * eps_b
        // a/b - a0/b0 = (a*b0 - a0*b)/(b*b0).
        div_cancel_common(a, b0, b)
        (a * b0) / (b * b0) = a / b
        a / b = (a * b0) / (b * b0)
        div_cancel_common(a0, b, b0)
        (a0 * b) / (b0 * b) = a0 / b0
        real_mul_comm(b0, b)
        b0 * b = b * b0
        (a0 * b) / (b * b0) = a0 / b0
        a0 / b0 = (a0 * b) / (b * b0)
        div_sub_same_denom(a, a0, b0, b, b * b0)
        a * b0 / (b * b0) - a0 * b / (b * b0) = (a * b0 - a0 * b) / (b * b0)
        a / b - a0 / b0 = (a * b0 - a0 * b) / (b * b0)
        // |a/b - a0/b0| = |a*b0 - a0*b| / (|b| * |b0|).
        abs_div(a * b0 - a0 * b, b * b0)
        ((a * b0 - a0 * b) / (b * b0)).abs =
            (a * b0 - a0 * b).abs / (b * b0).abs
        mul_not_zero[Real](b, b0)
        b * b0 != Real.0
        mul_abs(b, b0)
        b.abs * b0.abs = (b * b0).abs
        (b * b0).abs = b.abs * b0.abs
        (a / b - a0 / b0).abs = (a * b0 - a0 * b).abs / (b.abs * b0.abs)
        // |num| / (|b|*|b0|) < S / (|b|*|b0|) < S / ((|b0|/2)*|b0|).
        abs_gte_zero(a - a0)
        (a - a0).abs >= Real.0
        (a - a0).abs < eps_a
        lte_lt_trans(Real.0, (a - a0).abs, eps_a)
        Real.0 < eps_a
        gt_zero_imp_pos(eps_a)
        eps_a.is_positive
        pos_gt_zero(eps_a)
        eps_a > Real.0
        // S > 0: eps_a*b0.abs > 0 and a0.abs*eps_b >= 0.
        mul_pos_pos(eps_a, b0.abs)
        (eps_a * b0.abs).is_positive
        gt_zero_imp_pos(eps_a * b0.abs)
        eps_a * b0.abs > Real.0
        pos_gt_zero(eps_b)
        eps_b > Real.0
        lt_imp_lte(Real.0, eps_b)
        Real.0 <= eps_b
        mul_nonneg(a0.abs, eps_b)
        a0.abs * eps_b >= Real.0
        add_nonneg_pos_pos(a0.abs * eps_b, eps_a * b0.abs)
        a0.abs * eps_b + eps_a * b0.abs > Real.0
        real_mul_comm(eps_a, b0.abs)
        eps_a * b0.abs = b0.abs * eps_a
        add_comm(a0.abs * eps_b, eps_a * b0.abs)
        a0.abs * eps_b + eps_a * b0.abs = eps_a * b0.abs + a0.abs * eps_b
        eps_a * b0.abs + a0.abs * eps_b > Real.0
        (eps_a * b0.abs + a0.abs * eps_b).is_positive
        pos_gt_zero(eps_a * b0.abs + a0.abs * eps_b)
        div_le_div_pos((a * b0 - a0 * b).abs, eps_a * b0.abs + a0.abs * eps_b,
            b.abs * b0.abs)
        (a * b0 - a0 * b).abs / (b.abs * b0.abs) <= (eps_a * b0.abs + a0.abs * eps_b) / (b.abs * b0.abs)
        (a / b - a0 / b0).abs <= (eps_a * b0.abs + a0.abs * eps_b) / (b.abs * b0.abs)
        // 1/(|b|*|b0|) < 1/((|b0|/2)*|b0|).
        mul_pos_pos(b0.abs, b0.abs * Real.one_half)
        (b0.abs * (b0.abs * Real.one_half)).is_positive
        gt_zero_imp_pos(b0.abs * (b0.abs * Real.one_half))
        b0.abs * (b0.abs * Real.one_half) > Real.0
        inverse_on_positive_flips_inequality(b0.abs * (b0.abs * Real.one_half),
            b.abs * b0.abs)
        (b.abs * b0.abs).inverse < (b0.abs * (b0.abs * Real.one_half)).inverse
        mul_lt_mul_of_pos_right[Real]((b.abs * b0.abs).inverse,
            (b0.abs * (b0.abs * Real.one_half)).inverse,
            eps_a * b0.abs + a0.abs * eps_b)
        (eps_a * b0.abs + a0.abs * eps_b) * (b.abs * b0.abs).inverse < (eps_a * b0.abs + a0.abs * eps_b) *
                (b0.abs * (b0.abs * Real.one_half)).inverse
        // S / (|b|*|b0|) < S * ((|b0|/2)*|b0|).inverse.
        (eps_a * b0.abs + a0.abs * eps_b) / (b.abs * b0.abs) =
            (eps_a * b0.abs + a0.abs * eps_b) * (b.abs * b0.abs).inverse
        (eps_a * b0.abs + a0.abs * eps_b) / (b.abs * b0.abs) < (eps_a * b0.abs + a0.abs * eps_b) *
                (b0.abs * (b0.abs * Real.one_half)).inverse
        lte_lt_trans((a / b - a0 / b0).abs,
            (eps_a * b0.abs + a0.abs * eps_b) / (b.abs * b0.abs),
            (eps_a * b0.abs + a0.abs * eps_b) *
                (b0.abs * (b0.abs * Real.one_half)).inverse)
        (a / b - a0 / b0).abs < (eps_a * b0.abs + a0.abs * eps_b) *
                (b0.abs * (b0.abs * Real.one_half)).inverse
        // (|b0|/2)*|b0| = |b0|^2 * one_half.
        mul_assoc(b0.abs, b0.abs, Real.one_half)
        (b0.abs * b0.abs) * Real.one_half = b0.abs * (b0.abs * Real.one_half)
        b0.abs * (b0.abs * Real.one_half) = (b0.abs * b0.abs) * Real.one_half
        // one_half.inverse = two.
        mul_distrib_left(Real.one_half, Real.1, Real.1)
        (Real.1 + Real.1) * Real.one_half =
            Real.1 * Real.one_half + Real.1 * Real.one_half
        two * Real.one_half = Real.1 * Real.one_half + Real.1 * Real.one_half
        mul_one_left(Real.one_half)
        Real.1 * Real.one_half = Real.one_half
        two * Real.one_half = Real.one_half + Real.one_half
        one_half_plus_one_half
        Real.one_half + Real.one_half = Real.1
        two * Real.one_half = Real.1
        real_mul_comm(Real.one_half, two)
        Real.one_half * two = two * Real.one_half
        Real.one_half * two = Real.1
        lt_imp_ne(Real.0, Real.one_half)
        Real.one_half != Real.0
        mul_left_cancel(Real.1, Real.one_half, two)
        Real.1 / Real.one_half = two
        Real.one_half.inverse = Real.1 / Real.one_half
        Real.one_half.inverse = two
        // ((|b0|^2) * one_half).inverse = (|b0|^2).inverse * two.
        inverse_dist[Real](b0.abs * b0.abs, Real.one_half)
        ((b0.abs * b0.abs) * Real.one_half).inverse =
            (b0.abs * b0.abs).inverse * Real.one_half.inverse
        (b0.abs * (b0.abs * Real.one_half)).inverse =
            (b0.abs * b0.abs).inverse * Real.one_half.inverse
        (b0.abs * (b0.abs * Real.one_half)).inverse =
            (b0.abs * b0.abs).inverse * two
        // S * ((|b0|/2)*|b0|).inverse = (S * (|b0|^2).inverse) * two.
        mul_assoc(eps_a * b0.abs + a0.abs * eps_b,
            (b0.abs * b0.abs).inverse, two)
        ((eps_a * b0.abs + a0.abs * eps_b) * (b0.abs * b0.abs).inverse) * two =
            (eps_a * b0.abs + a0.abs * eps_b) *
                ((b0.abs * b0.abs).inverse * two)
        (eps_a * b0.abs + a0.abs * eps_b) *
            (b0.abs * (b0.abs * Real.one_half)).inverse =
            ((eps_a * b0.abs + a0.abs * eps_b) * (b0.abs * b0.abs).inverse) * two
        // two * S / |b0|^2 < eps.
        mul_pos_pos(b0.abs, b0.abs)
        (b0.abs * b0.abs).is_positive
        gt_zero_imp_pos(b0.abs * b0.abs)
        b0.abs * b0.abs > Real.0
        two_positive
        two.is_positive
        gt_zero_imp_pos(two)
        two > Real.0
        mul_pos_pos(two, eps_a * b0.abs + a0.abs * eps_b)
        (two * (eps_a * b0.abs + a0.abs * eps_b)).is_positive
        gt_zero_imp_pos(two * (eps_a * b0.abs + a0.abs * eps_b))
        two * (eps_a * b0.abs + a0.abs * eps_b) > Real.0
        lt_mul_pos_imp_div_lt(two * (eps_a * b0.abs + a0.abs * eps_b),
            b0.abs * b0.abs, eps)
        (two * (eps_a * b0.abs + a0.abs * eps_b)) / (b0.abs * b0.abs) < eps
        // (S * (|b0|^2).inverse) * two = two * S / |b0|^2.
        real_mul_comm(two, (b0.abs * b0.abs).inverse)
        two * (b0.abs * b0.abs).inverse = (b0.abs * b0.abs).inverse * two
        mul_assoc(two, eps_a * b0.abs + a0.abs * eps_b, (b0.abs * b0.abs).inverse)
        (two * (eps_a * b0.abs + a0.abs * eps_b)) * (b0.abs * b0.abs).inverse =
            two * ((eps_a * b0.abs + a0.abs * eps_b) * (b0.abs * b0.abs).inverse)
        real_mul_comm(eps_a * b0.abs + a0.abs * eps_b, (b0.abs * b0.abs).inverse)
        (eps_a * b0.abs + a0.abs * eps_b) * (b0.abs * b0.abs).inverse =
            (b0.abs * b0.abs).inverse * (eps_a * b0.abs + a0.abs * eps_b)
        ((eps_a * b0.abs + a0.abs * eps_b) * (b0.abs * b0.abs).inverse) * two =
            (two * (eps_a * b0.abs + a0.abs * eps_b)) * (b0.abs * b0.abs).inverse
        (two * (eps_a * b0.abs + a0.abs * eps_b)) / (b0.abs * b0.abs) =
            (two * (eps_a * b0.abs + a0.abs * eps_b)) * (b0.abs * b0.abs).inverse
        (eps_a * b0.abs + a0.abs * eps_b) *
            (b0.abs * (b0.abs * Real.one_half)).inverse =
            (two * (eps_a * b0.abs + a0.abs * eps_b)) / (b0.abs * b0.abs)
        (a / b - a0 / b0).abs < (two * (eps_a * b0.abs + a0.abs * eps_b)) / (b0.abs * b0.abs)
        lt_trans[Real]((a / b - a0 / b0).abs,
            (two * (eps_a * b0.abs + a0.abs * eps_b)) / (b0.abs * b0.abs), eps)
        (a / b - a0 / b0).abs < eps
    }
}

// =============================================================================
// The implicit function theorem (statement, proof deferred)
// =============================================================================

/// The derivative of a function g defined implicitly by f(x, g(x)) = 0 with
/// g(x0) = y0: under the standard hypotheses (the partial derivatives exist
/// and are continuous at (x0, y0), fy(x0, y0) != 0, and g is continuous at
/// x0), g is differentiable at x0 with derivative -fx(x0, y0) / fy(x0, y0).
///
/// The proof differentiates the identity f(x, g(x)) = 0 by applying the mean
/// value theorem to the two slices (see `mvt_x_slice` and `mvt_y_slice`):
/// for every x there are points c between x0 and x with
/// f(x, g(x)) - f(x0, g(x)) = fx(c, g(x)) * (x - x0), and d between y0 and
/// g(x) with f(x0, g(x)) - f(x0, y0) = fy(x0, d) * (g(x) - y0).  Adding the
/// two identities and dividing by fy(x0, d) * (x - x0) yields
/// (g(x) - y0) / (x - x0) = -fx(c, g(x)) / fy(x0, d), whose right-hand side
/// converges to -fx(x0, y0) / fy(x0, y0) by continuity of the partial
/// derivatives (see `continuous_2` and `ratio_close`) and of g.  The
/// ε-δ bookkeeping of this convergence argument is left as future work.
// theorem implicit_function_derivative(f: Real -> Real -> Real,
//     fx: Real -> Real -> Real, fy: Real -> Real -> Real,
//     x0: Real, y0: Real, g: Real -> Real) {
//     (forall(x: Real) { f(x, g(x)) = Real.0 }) and f(x0, y0) = Real.0 and g(x0) = y0 and
//     (forall(y: Real) { continuous(partial_x_fn(f, y)) }) and
//     (forall(y: Real) { is_derivative_fn(partial_x_fn(f, y), partial_x_fn(fx, y)) }) and
//     (forall(x: Real) { continuous(partial_y_fn(f, x)) }) and
//     (forall(x: Real) { is_derivative_fn(partial_y_fn(f, x), partial_y_fn(fy, x)) }) and
//     fy(x0, y0) != Real.0 and
//     continuous_at(g, x0) and
//     continuous_2(fx, x0, y0) and continuous_at(partial_y_fn(fy, x0), y0)
//     implies has_derivative_at(g, x0, -(fx(x0, y0)) / fy(x0, y0))
// }
