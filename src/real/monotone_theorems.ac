/// Monotone function theorems and the inverse function theorem.
///
/// This file develops the capstone results of first-course real analysis for
/// monotone functions on closed intervals:
///   * a continuous injective function on an interval is strictly monotone,
///   * a nondecreasing function on a closed interval attains its extrema at
///     the endpoints,
///   * a strictly positive derivative implies strict increase,
///   * the inverse function theorem: a continuous strictly increasing function
///     on [a, b] has a continuous strictly increasing inverse on the range
///     interval [f(a), f(b)] whose derivative is the reciprocal derivative
///     where the derivative of f is nonzero.

from order import lte_refl, lte_trans, lt_trans, lt_imp_lte, not_lt_imp_gte, lte_antisymm,
    lt_of_lte_of_lt, lt_of_lt_of_lte, lt_imp_ne, lt_imp_ne_symm, not_lt_self, not_lte_imp_gt,
    lt_of_lte_of_ne, lt_of_lte_of_ne_symm
from order_set import closed_interval_set, closed_interval_set_contains_eq,
    closed_interval_set_lower_le, closed_interval_set_le_upper,
    closed_interval_set_contains_lower, closed_interval_set_contains_upper,
    closed_interval_set_subset_of_bounds
from order import closed_interval
from data.basic.set import Set, subset_contains
from data.basic.function_algebra import pointwise_neg
from algebra.add_ordered_group import neg_lt_neg, neg_le_neg, le_of_neg_le_neg
from data.basic.witness import choose_or_default, choose_or_default_spec,
    choose_or_default_unique_eq, exists_unique, exists_unique_exists,
    exists_unique_of_exists_and_pairwise, exists_unique_intro
from real.continuity_base import Real, continuous
from real.intermediate_value import intermediate_value_closed_interval
from real.mean_value import mean_value_theorem, secant_slope,
    nonneg_derivative_imp_nondecreasing
from real.derivative_basic import has_derivative_at, has_derivative_at_unique,
    difference_quotient, sub_ne_zero_of_ne
from real.calculus_api import is_derivative_fn, is_derivative_fn_at
from real.uniform_continuity import continuous_on, continuous_on_at,
    continuous_on_condition
from real.real_base import add_comm, close_imp_bounds, lt_add_pos, pos_imp_eq_abs,
    neg_zero, neg_distrib, add_assoc, self_close, sub_cancels, neg_neg, abs_neg,
    lt_add_right, lte_add_right, add_neg_eq_zero, gt_zero_imp_pos, add_zero_left,
    add_zero_right, neg_pos_is_neg, neg_lt_zero, pos_gt_zero, lt_add_converse,
    bounds_imp_close, lte_abs, abs_gte_zero, lte_lt_trans
from real.real_seq import eps_smaller_than_both, smaller_pos,
    close_and_lt_imp_close, sub_zero_imp_eq, lt_imp_minus_pos, neg_is_close
from real.real_ring import mul_zero_left, mul_zero_right,
    pos_lte_imp_pos, real_mul_comm, mul_distrib_right, mul_neg_left, mul_neg_right,
    mul_one_right, mul_one_left
from ordered_field import inverse_of_positive_is_positive, inverse_of_negative_is_negative,
    mul_le_mul_of_nonpos_right, multiply_inequality_with_nonnegative_element,
    mul_lt_mul_of_pos_right, mul_le_mul_of_nonneg_right
from real.derivative_continuity import div_mul_cancel_denominator
from real.continuity_pointwise import continuous_pointwise_neg
from real.integral_exp import add_sub_cancel_shift
from real.monotone_betweenness import continuous_injective_betweenness_rising,
    continuous_injective_betweenness_falling
from real.monotone_base import increasing_on, decreasing_on,
    nondecreasing_on, nonincreasing_on, injective_on, increasing_on_apply,
    nondecreasing_on_apply, nonincreasing_on_apply, injective_on_apply,
    increasing_on_values_imp_args_eq, intermediate_value_closed_interval_rev,
    closed_interval_inner_member, increasing_on_values_imp_args_lt,
    injective_on_pointwise_neg, closed_interval_set_contains_of_bounds

numerals Real



// =============================================================================
// Continuous injective functions are strictly monotone
// =============================================================================

/// A continuous injective function with the left endpoint value below the right
/// endpoint value keeps every value between the endpoint values.
theorem continuous_injective_between_endpoint_values(
    f: Real -> Real, a: Real, b: Real, u: Real
) {
    continuous(f) and injective_on(f, a, b) and f(a) < f(b) and
    closed_interval_set(a, b).contains(u)
    implies f(a) <= f(u) and f(u) <= f(b)
} by {
    if continuous(f) and injective_on(f, a, b) and f(a) < f(b) and
       closed_interval_set(a, b).contains(u) {
        closed_interval_set_lower_le(a, b, u)
        a <= u
        closed_interval_set_le_upper(a, b, u)
        u <= b
        lte_trans[Real](a, u, b)
        a <= b
        if u = a {
            lt_imp_lte(f(a), f(b))
            f(a) <= f(b)
            u = a
            f(u) = f(a)
            lte_refl(f(a))
            f(a) <= f(a)
            f(a) <= f(u)
            f(u) <= f(b)
            f(a) <= f(u) and f(u) <= f(b)
        } else {
            if u = b {
                lt_imp_lte(f(a), f(b))
                f(a) <= f(b)
                u = b
                f(u) = f(b)
                lte_refl(f(b))
                f(b) <= f(b)
                f(a) <= f(u)
                f(u) <= f(b)
                f(a) <= f(u) and f(u) <= f(b)
            } else {
                lt_of_lte_of_ne(a, u)
                a < u
                lt_of_lte_of_ne_symm(u, b)
                u < b
                if f(u) < f(a) {
                    lt_imp_lte(f(u), f(a))
                    f(u) <= f(a)
                    lt_imp_lte(f(a), f(b))
                    f(a) <= f(b)
                    intermediate_value_closed_interval(f, u, b, f(a))
                    let w: Real satisfy {
                        closed_interval_set(u, b).contains(w) and f(w) = f(a)
                    }
                    closed_interval_set(u, b).contains(w)
                    closed_interval_set_lower_le(u, b, w)
                    u <= w
                    lt_of_lt_of_lte[Real](a, u, w)
                    a < w
                    lt_imp_ne_symm(a, w)
                    w != a
                    closed_interval_set_le_upper(u, b, w)
                    w <= b
                    lt_imp_lte(a, w)
                    a <= w
                    closed_interval(a, b, w)
                    closed_interval_set_contains_eq(a, b, w)
                    closed_interval_set(a, b).contains(w)
                    closed_interval_set_contains_lower(a, b)
                    closed_interval_set(a, b).contains(a)
                    f(w) = f(a)
                    injective_on_apply(f, a, b, w, a)
                    w = a
                    false
                }
                not_lt_imp_gte[Real](f(u), f(a))
                f(a) <= f(u)
                if f(b) < f(u) {
                    lt_imp_lte(f(a), f(b))
                    f(a) <= f(b)
                    lt_imp_lte(f(b), f(u))
                    f(b) <= f(u)
                    intermediate_value_closed_interval(f, a, u, f(b))
                    let w: Real satisfy {
                        closed_interval_set(a, u).contains(w) and f(w) = f(b)
                    }
                    closed_interval_set(a, u).contains(w)
                    closed_interval_set_le_upper(a, u, w)
                    w <= u
                    lt_of_lte_of_lt[Real](w, u, b)
                    w < b
                    lt_imp_ne(w, b)
                    w != b
                    closed_interval_set_lower_le(a, u, w)
                    a <= w
                    lte_trans[Real](w, u, b)
                    w <= b
                    closed_interval(a, b, w)
                    closed_interval_set_contains_eq(a, b, w)
                    closed_interval_set(a, b).contains(w)
                    closed_interval_set_contains_upper(a, b)
                    closed_interval_set(a, b).contains(b)
                    f(w) = f(b)
                    injective_on_apply(f, a, b, w, b)
                    w = b
                    false
                }
                not_lt_imp_gte[Real](f(b), f(u))
                f(u) <= f(b)
                f(a) <= f(u) and f(u) <= f(b)
            }
        }
    }
}

/// A continuous injective function with the right endpoint value below the left
/// endpoint value keeps every value between the endpoint values.
theorem continuous_injective_between_endpoint_values_rev(
    f: Real -> Real, a: Real, b: Real, u: Real
) {
    continuous(f) and injective_on(f, a, b) and f(b) < f(a) and
    closed_interval_set(a, b).contains(u)
    implies f(b) <= f(u) and f(u) <= f(a)
} by {
    if continuous(f) and injective_on(f, a, b) and f(b) < f(a) and
       closed_interval_set(a, b).contains(u) {
        continuous_pointwise_neg(f)
        continuous(pointwise_neg(f))
        injective_on_pointwise_neg(f, a, b)
        injective_on(pointwise_neg(f), a, b)
        neg_lt_neg[Real](f(b), f(a))
        -f(a) < -f(b)
        pointwise_neg(f)(a) = -f(a)
        pointwise_neg(f)(b) = -f(b)
        pointwise_neg(f)(a) < pointwise_neg(f)(b)
        continuous_injective_between_endpoint_values(pointwise_neg(f), a, b, u)
        pointwise_neg(f)(a) <= pointwise_neg(f)(u) and
            pointwise_neg(f)(u) <= pointwise_neg(f)(b)
        pointwise_neg(f)(a) <= pointwise_neg(f)(u)
        pointwise_neg(f)(u) <= pointwise_neg(f)(b)
        pointwise_neg(f)(u) = -f(u)
        pointwise_neg(f)(a) = -f(a)
        pointwise_neg(f)(b) = -f(b)
        -f(a) <= -f(u)
        -f(u) <= -f(b)
        le_of_neg_le_neg[Real](f(u), f(a))
        f(u) <= f(a)
        le_of_neg_le_neg[Real](f(b), f(u))
        f(b) <= f(u)
        f(b) <= f(u) and f(u) <= f(a)
    }
}

/// A continuous injective function with the left endpoint value below the right
/// endpoint value is strictly increasing on the interval.
theorem continuous_injective_imp_strictly_increasing(f: Real -> Real, a: Real, b: Real) {
    continuous(f) and a < b and injective_on(f, a, b) and f(a) < f(b)
    implies increasing_on(f, a, b)
} by {
    if continuous(f) and a < b and injective_on(f, a, b) and f(a) < f(b) {
        forall(x: Real, y: Real) {
            if closed_interval_set(a, b).contains(x) and
               closed_interval_set(a, b).contains(y) and
               x < y {
                lt_imp_lte(a, b)
                a <= b
                closed_interval_set_contains_lower(a, b)
                closed_interval_set(a, b).contains(a)
                closed_interval_set_lower_le(a, b, x)
                a <= x
                continuous_injective_between_endpoint_values(f, a, b, x)
                f(a) <= f(x) and f(x) <= f(b)
                f(a) <= f(x)
                continuous_injective_between_endpoint_values(f, a, b, y)
                f(a) <= f(y) and f(y) <= f(b)
                f(a) <= f(y)
                if a < x {
                    if f(x) = f(a) {
                        injective_on_apply(f, a, b, x, a)
                        x = a
                        false
                    }
                    f(x) != f(a)
                    lt_of_lte_of_ne_symm(f(a), f(x))
                    f(a) < f(x)
                    continuous_injective_betweenness_rising(f, a, b, a, x, y)
                    f(x) < f(y)
                } else {
                    not_lt_imp_gte[Real](a, x)
                    a >= x
                    x <= a
                    lte_antisymm[Real](a, x)
                    a = x
                    x = a
                    f(x) = f(a)
                    lt_of_lte_of_lt[Real](a, x, y)
                    a < y
                    lt_imp_ne_symm(a, y)
                    y != a
                    if f(y) = f(a) {
                        injective_on_apply(f, a, b, y, a)
                        closed_interval_set(a, b).contains(y)
                        y = a
                        false
                    }
                    f(y) != f(a)
                    lt_of_lte_of_ne_symm(f(a), f(y))
                    f(a) < f(y)
                    f(a) = f(x)
                    f(x) < f(y)
                }
            }
        }
        increasing_on(f, a, b)
    }
}

/// A continuous injective function with the right endpoint value below the left
/// endpoint value is strictly decreasing on the interval.
theorem continuous_injective_imp_strictly_decreasing(f: Real -> Real, a: Real, b: Real) {
    continuous(f) and a < b and injective_on(f, a, b) and f(b) < f(a)
    implies decreasing_on(f, a, b)
} by {
    if continuous(f) and a < b and injective_on(f, a, b) and f(b) < f(a) {
        forall(x: Real, y: Real) {
            if closed_interval_set(a, b).contains(x) and
               closed_interval_set(a, b).contains(y) and
               x < y {
                lt_imp_lte(a, b)
                a <= b
                closed_interval_set_contains_upper(a, b)
                closed_interval_set(a, b).contains(b)
                closed_interval_set_le_upper(a, b, y)
                y <= b
                continuous_injective_between_endpoint_values_rev(f, a, b, x)
                f(b) <= f(x) and f(x) <= f(a)
                f(b) <= f(x)
                continuous_injective_between_endpoint_values_rev(f, a, b, y)
                f(b) <= f(y) and f(y) <= f(a)
                f(b) <= f(y)
                if y < b {
                    if f(y) = f(b) {
                        injective_on_apply(f, a, b, y, b)
                        y = b
                        false
                    }
                    f(y) != f(b)
                    lt_of_lte_of_ne_symm(f(b), f(y))
                    f(b) < f(y)
                    continuous_injective_betweenness_falling(f, a, b, x, y, b)
                    f(y) < f(x)
                } else {
                    not_lt_imp_gte[Real](y, b)
                    y >= b
                    b <= y
                    lte_antisymm[Real](y, b)
                    y = b
                    f(y) = f(b)
                    lt_of_lte_of_lt[Real](x, y, b)
                    x < b
                    lt_imp_ne(x, b)
                    b != x
                    x != b
                    if f(x) = f(b) {
                        injective_on_apply(f, a, b, x, b)
                        x = b
                        false
                    }
                    f(x) != f(b)
                    lt_of_lte_of_ne_symm(f(b), f(x))
                    f(b) < f(x)
                    f(b) = f(y)
                    f(y) < f(x)
                }
            }
        }
        decreasing_on(f, a, b)
    }
}

// =============================================================================
// Monotone functions attain their extrema at the endpoints
// =============================================================================

/// A nondecreasing function on a closed interval is bounded below by its value
/// at the left endpoint and above by its value at the right endpoint.
theorem nondecreasing_on_endpoint_extrema(f: Real -> Real, a: Real, b: Real, x: Real) {
    nondecreasing_on(f, a, b) and closed_interval_set(a, b).contains(x)
    implies f(a) <= f(x) and f(x) <= f(b)
} by {
    if nondecreasing_on(f, a, b) and closed_interval_set(a, b).contains(x) {
        closed_interval_set_lower_le(a, b, x)
        a <= x
        closed_interval_set_le_upper(a, b, x)
        x <= b
        lte_trans[Real](a, x, b)
        a <= b
        closed_interval_set_contains_lower(a, b)
        closed_interval_set(a, b).contains(a)
        closed_interval_set_contains_upper(a, b)
        closed_interval_set(a, b).contains(b)
        nondecreasing_on_apply(f, a, b, a, x)
        f(a) <= f(x)
        nondecreasing_on_apply(f, a, b, x, b)
        f(x) <= f(b)
        f(a) <= f(x) and f(x) <= f(b)
    }
}

/// A nonincreasing function on a closed interval is bounded above by its value
/// at the left endpoint and below by its value at the right endpoint.
theorem nonincreasing_on_endpoint_extrema(f: Real -> Real, a: Real, b: Real, x: Real) {
    nonincreasing_on(f, a, b) and closed_interval_set(a, b).contains(x)
    implies f(x) <= f(a) and f(b) <= f(x)
} by {
    if nonincreasing_on(f, a, b) and closed_interval_set(a, b).contains(x) {
        closed_interval_set_lower_le(a, b, x)
        a <= x
        closed_interval_set_le_upper(a, b, x)
        x <= b
        lte_trans[Real](a, x, b)
        a <= b
        closed_interval_set_contains_lower(a, b)
        closed_interval_set(a, b).contains(a)
        closed_interval_set_contains_upper(a, b)
        closed_interval_set(a, b).contains(b)
        nonincreasing_on_apply(f, a, b, a, x)
        f(x) <= f(a)
        nonincreasing_on_apply(f, a, b, x, b)
        f(b) <= f(x)
        f(x) <= f(a) and f(b) <= f(x)
    }
}

// =============================================================================
// Strictly positive derivative implies strict increase
// =============================================================================

/// A function whose derivative is everywhere strictly positive is strictly
/// increasing on every interval.
theorem positive_derivative_imp_strictly_increasing(
    f: Real -> Real, df: Real -> Real, a: Real, b: Real, x: Real, y: Real
) {
    continuous(f) and a < b and is_derivative_fn(f, df) and
    (forall(z: Real) { Real.0 < df(z) }) and
    closed_interval_set(a, b).contains(x) and closed_interval_set(a, b).contains(y) and
    x < y
    implies f(x) < f(y)
} by {
    if continuous(f) and a < b and is_derivative_fn(f, df) and
       (forall(z: Real) { Real.0 < df(z) }) and
       closed_interval_set(a, b).contains(x) and closed_interval_set(a, b).contains(y) and
       x < y {
        if x < y {
            mean_value_theorem(f, df, x, y)
            let c: Real satisfy {
                x < c and c < y and has_derivative_at(f, c, secant_slope(f, x, y))
            }
            x < c and c < y
            has_derivative_at(f, c, secant_slope(f, x, y))
            is_derivative_fn_at(f, df, c)
            has_derivative_at(f, c, df(c))
            has_derivative_at_unique(f, c, secant_slope(f, x, y), df(c))
            secant_slope(f, x, y) = df(c)
            forall(z: Real) { Real.0 < df(z) }
            Real.0 < df(c)
            Real.0 < secant_slope(f, x, y)
            secant_slope(f, x, y) = (f(y) - f(x)) / (y - x)
            Real.0 < (f(y) - f(x)) / (y - x)
            lt_imp_minus_pos(x, y)
            (y - x).is_positive
            pos_gt_zero(y - x)
            y - x > Real.0
            mul_lt_mul_of_pos_right[Real](Real.0, (f(y) - f(x)) / (y - x), y - x)
            Real.0 * (y - x) < ((f(y) - f(x)) / (y - x)) * (y - x)
            mul_zero_left(y - x)
            Real.0 * (y - x) = Real.0
            Real.0 < ((f(y) - f(x)) / (y - x)) * (y - x)
            lt_imp_ne_symm(x, y)
            y != x
            sub_ne_zero_of_ne(y, x)
            y - x != Real.0
            div_mul_cancel_denominator(f(y) - f(x), y - x)
            ((f(y) - f(x)) / (y - x)) * (y - x) = f(y) - f(x)
            Real.0 < f(y) - f(x)
            lt_add_right(Real.0, f(y) - f(x), f(x))
            Real.0 + f(x) < (f(y) - f(x)) + f(x)
            add_zero_left(f(x))
            Real.0 + f(x) = f(x)
            (f(y) - f(x)) + f(x) = f(y)
            f(x) < f(y)
        }
    }
}

// =============================================================================
// The inverse of a strictly increasing continuous function
// =============================================================================

/// True if x is a preimage of y under f lying in [a, b].
define f_inv_pred(f: Real -> Real, a: Real, b: Real, y: Real, x: Real) -> Bool {
    closed_interval_set(a, b).contains(x) and f(x) = y
}

/// A preimage membership is the defining conjunction.
theorem f_inv_pred_contains_eq(f: Real -> Real, a: Real, b: Real, y: Real, x: Real) {
    f_inv_pred(f, a, b, y, x) = (closed_interval_set(a, b).contains(x) and f(x) = y)
} by {
    f_inv_pred(f, a, b, y, x) = (closed_interval_set(a, b).contains(x) and f(x) = y)
}

/// The inverse of a strictly increasing continuous function on [a, b]: the
/// unique preimage in [a, b] when y lies in the range, and the value a
/// otherwise.
define f_inv_of(f: Real -> Real, a: Real, b: Real, y: Real) -> Real {
    choose_or_default(f_inv_pred(f, a, b, y), a)
}

/// Every value in [f(a), f(b)] has a unique preimage in [a, b].
theorem f_inv_unique(f: Real -> Real, a: Real, b: Real, y: Real) {
    continuous(f) and a < b and increasing_on(f, a, b) and
    closed_interval_set(f(a), f(b)).contains(y)
    implies exists_unique(f_inv_pred(f, a, b, y))
} by {
    if continuous(f) and a < b and increasing_on(f, a, b) and
       closed_interval_set(f(a), f(b)).contains(y) {
        closed_interval_set_lower_le(f(a), f(b), y)
        f(a) <= y
        closed_interval_set_le_upper(f(a), f(b), y)
        y <= f(b)
        lt_imp_lte(a, b)
        a <= b
        intermediate_value_closed_interval(f, a, b, y)
        let x0: Real satisfy {
            closed_interval_set(a, b).contains(x0) and f(x0) = y
        }
        f_inv_pred_contains_eq(f, a, b, y, x0)
        f_inv_pred(f, a, b, y, x0)
        exists(w: Real) {
            f_inv_pred(f, a, b, y, w)
        }
        forall(w: Real) {
            if f_inv_pred(f, a, b, y, w) {
                f_inv_pred_contains_eq(f, a, b, y, w)
                closed_interval_set(a, b).contains(w) and f(w) = y
                closed_interval_set(a, b).contains(w)
                f(w) = y
                f(x0) = y
                f(w) = f(x0)
                closed_interval_set(a, b).contains(x0)
                closed_interval_set(a, b).contains(w)
                increasing_on_values_imp_args_eq(f, a, b, w, x0)
                w = x0
            }
        }
        exists_unique_intro(f_inv_pred(f, a, b, y), x0)
        exists_unique(f_inv_pred(f, a, b, y))
    }
}

/// The inverse recovers the argument of f on the range interval (right inverse).
theorem f_inv_spec(f: Real -> Real, a: Real, b: Real, y: Real) {
    continuous(f) and a < b and increasing_on(f, a, b) and
    closed_interval_set(f(a), f(b)).contains(y)
    implies closed_interval_set(a, b).contains(f_inv_of(f, a, b, y)) and
            f(f_inv_of(f, a, b, y)) = y
} by {
    if continuous(f) and a < b and increasing_on(f, a, b) and
       closed_interval_set(f(a), f(b)).contains(y) {
        f_inv_unique(f, a, b, y)
        exists_unique(f_inv_pred(f, a, b, y))
        exists_unique_exists(f_inv_pred(f, a, b, y))
        exists(x: Real) {
            f_inv_pred(f, a, b, y, x)
        }
        choose_or_default_spec(f_inv_pred(f, a, b, y), a)
        f_inv_pred(f, a, b, y, f_inv_of(f, a, b, y))
        f_inv_pred_contains_eq(f, a, b, y, f_inv_of(f, a, b, y))
        closed_interval_set(a, b).contains(f_inv_of(f, a, b, y)) and
            f(f_inv_of(f, a, b, y)) = y
        closed_interval_set(a, b).contains(f_inv_of(f, a, b, y))
        f(f_inv_of(f, a, b, y)) = y
        closed_interval_set(a, b).contains(f_inv_of(f, a, b, y)) and
            f(f_inv_of(f, a, b, y)) = y
    }
}

/// The inverse recovers the argument of f on [a, b] (left inverse).
theorem f_inv_left_inverse(f: Real -> Real, a: Real, b: Real, x: Real) {
    continuous(f) and a < b and increasing_on(f, a, b) and
    closed_interval_set(a, b).contains(x)
    implies f_inv_of(f, a, b, f(x)) = x
} by {
    if continuous(f) and a < b and increasing_on(f, a, b) and
       closed_interval_set(a, b).contains(x) {
        closed_interval_set_lower_le(a, b, x)
        a <= x
        closed_interval_set_le_upper(a, b, x)
        x <= b
        lt_imp_lte(a, b)
        a <= b
        closed_interval_set_contains_lower(a, b)
        closed_interval_set(a, b).contains(a)
        closed_interval_set_contains_upper(a, b)
        closed_interval_set(a, b).contains(b)
        if a < x {
            increasing_on_apply(f, a, b, a, x)
            f(a) < f(x)
            lt_imp_lte(f(a), f(x))
            f(a) <= f(x)
        } else {
            not_lt_imp_gte[Real](a, x)
            a >= x
            x <= a
            lte_antisymm[Real](a, x)
            a = x
            f(a) = f(x)
            lt_imp_lte(f(a), f(x))
            f(a) <= f(x)
        }
        if x < b {
            increasing_on_apply(f, a, b, x, b)
            f(x) < f(b)
            lt_imp_lte(f(x), f(b))
            f(x) <= f(b)
        } else {
            not_lt_imp_gte[Real](x, b)
            x >= b
            b <= x
            lte_antisymm[Real](x, b)
            x = b
            f(x) = f(b)
            lt_imp_lte(f(x), f(b))
            f(x) <= f(b)
        }
        closed_interval(f(a), f(b), f(x))
        closed_interval_set_contains_eq(f(a), f(b), f(x))
        closed_interval_set(f(a), f(b)).contains(f(x))
        f_inv_unique(f, a, b, f(x))
        exists_unique(f_inv_pred(f, a, b, f(x)))
        f_inv_pred_contains_eq(f, a, b, f(x), x)
        f_inv_pred(f, a, b, f(x), x)
        choose_or_default_unique_eq(f_inv_pred(f, a, b, f(x)), a, x)
        choose_or_default(f_inv_pred(f, a, b, f(x)), a) = x
        f_inv_of(f, a, b, f(x)) = x
    }
}

/// The inverse is strictly increasing on the range interval.
theorem f_inv_strictly_increasing(f: Real -> Real, a: Real, b: Real, y1: Real, y2: Real) {
    continuous(f) and a < b and increasing_on(f, a, b) and
    closed_interval_set(f(a), f(b)).contains(y1) and
    closed_interval_set(f(a), f(b)).contains(y2) and
    y1 < y2
    implies f_inv_of(f, a, b, y1) < f_inv_of(f, a, b, y2)
} by {
    if continuous(f) and a < b and increasing_on(f, a, b) and
       closed_interval_set(f(a), f(b)).contains(y1) and
       closed_interval_set(f(a), f(b)).contains(y2) and
       y1 < y2 {
        f_inv_spec(f, a, b, y1)
        closed_interval_set(a, b).contains(f_inv_of(f, a, b, y1))
        f(f_inv_of(f, a, b, y1)) = y1
        f_inv_spec(f, a, b, y2)
        closed_interval_set(a, b).contains(f_inv_of(f, a, b, y2))
        f(f_inv_of(f, a, b, y2)) = y2
        if not f_inv_of(f, a, b, y1) < f_inv_of(f, a, b, y2) {
            not_lt_imp_gte[Real](f_inv_of(f, a, b, y1), f_inv_of(f, a, b, y2))
            f_inv_of(f, a, b, y2) <= f_inv_of(f, a, b, y1)
            if f_inv_of(f, a, b, y2) < f_inv_of(f, a, b, y1) {
                increasing_on_apply(f, a, b, f_inv_of(f, a, b, y2), f_inv_of(f, a, b, y1))
                f(f_inv_of(f, a, b, y2)) < f(f_inv_of(f, a, b, y1))
                f(f_inv_of(f, a, b, y2)) = y2
                f(f_inv_of(f, a, b, y1)) = y1
                y2 < y1
                y1 < y2
                false
            } else {
                not_lt_imp_gte[Real](f_inv_of(f, a, b, y2), f_inv_of(f, a, b, y1))
                f_inv_of(f, a, b, y1) <= f_inv_of(f, a, b, y2)
                lte_antisymm[Real](f_inv_of(f, a, b, y1), f_inv_of(f, a, b, y2))
                f_inv_of(f, a, b, y1) = f_inv_of(f, a, b, y2)
                f(f_inv_of(f, a, b, y1)) = f(f_inv_of(f, a, b, y2))
                f(f_inv_of(f, a, b, y1)) = y1
                f(f_inv_of(f, a, b, y2)) = y2
                y1 = y2
                y1 < y2
                false
            }
        }
        f_inv_of(f, a, b, y1) < f_inv_of(f, a, b, y2)
    }
}

/// A value strictly between f(x_l) and f(x_r) has its preimage strictly
/// between x_l and x_r.
theorem f_inv_between_trap(f: Real -> Real, a: Real, b: Real, x_l: Real, x_r: Real, y1: Real) {
    continuous(f) and a < b and increasing_on(f, a, b) and
    closed_interval_set(a, b).contains(x_l) and closed_interval_set(a, b).contains(x_r) and
    x_l < x_r and
    closed_interval_set(f(a), f(b)).contains(y1) and
    f(x_l) < y1 and y1 < f(x_r)
    implies x_l < f_inv_of(f, a, b, y1) and f_inv_of(f, a, b, y1) < x_r
} by {
    if continuous(f) and a < b and increasing_on(f, a, b) and
       closed_interval_set(a, b).contains(x_l) and closed_interval_set(a, b).contains(x_r) and
       x_l < x_r and
       closed_interval_set(f(a), f(b)).contains(y1) and
       f(x_l) < y1 and y1 < f(x_r) {
        f_inv_spec(f, a, b, y1)
        closed_interval_set(a, b).contains(f_inv_of(f, a, b, y1))
        f(f_inv_of(f, a, b, y1)) = y1
        f(x_l) < f(f_inv_of(f, a, b, y1))
        increasing_on_values_imp_args_lt(f, a, b, x_l, f_inv_of(f, a, b, y1))
        x_l < f_inv_of(f, a, b, y1)
        f(f_inv_of(f, a, b, y1)) < f(x_r)
        increasing_on_values_imp_args_lt(f, a, b, f_inv_of(f, a, b, y1), x_r)
        f_inv_of(f, a, b, y1) < x_r
        x_l < f_inv_of(f, a, b, y1) and f_inv_of(f, a, b, y1) < x_r
    }
}

/// The inverse of a continuous strictly increasing function is continuous on
/// the range interval [f(a), f(b)].
theorem f_inv_continuous_on(f: Real -> Real, a: Real, b: Real) {
    continuous(f) and a < b and increasing_on(f, a, b)
    implies continuous_on(f_inv_of(f, a, b), f(a), f(b))
} by {
    if continuous(f) and a < b and increasing_on(f, a, b) {
        forall(y0: Real) {
            if closed_interval_set(f(a), f(b)).contains(y0) {
                forall(eps: Real) {
                    if eps.is_positive {
                        f_inv_spec(f, a, b, y0)
                        closed_interval_set(a, b).contains(f_inv_of(f, a, b, y0))
                        f(f_inv_of(f, a, b, y0)) = y0
                        closed_interval_set_lower_le(a, b, f_inv_of(f, a, b, y0))
                        a <= f_inv_of(f, a, b, y0)
                        closed_interval_set_le_upper(a, b, f_inv_of(f, a, b, y0))
                        f_inv_of(f, a, b, y0) <= b
                        if a < f_inv_of(f, a, b, y0) {
                            if f_inv_of(f, a, b, y0) < b {
                                // ===== interior point: y0 in (f(a), f(b)) =====
                                lt_imp_minus_pos(a, f_inv_of(f, a, b, y0))
                                (f_inv_of(f, a, b, y0) - a).is_positive
                                lt_imp_minus_pos(f_inv_of(f, a, b, y0), b)
                                (b - f_inv_of(f, a, b, y0)).is_positive
                                eps_smaller_than_both(eps, f_inv_of(f, a, b, y0) - a)
                                let eta1: Real satisfy {
                                    eta1.is_positive and eta1 < eps and
                                    eta1 < f_inv_of(f, a, b, y0) - a
                                }
                                eps_smaller_than_both(eta1, b - f_inv_of(f, a, b, y0))
                                let eta: Real satisfy {
                                    eta.is_positive and eta < eta1 and
                                    eta < b - f_inv_of(f, a, b, y0)
                                }
                                lt_trans[Real](eta, eta1, eps)
                                eta < eps
                                lt_trans[Real](eta, eta1, f_inv_of(f, a, b, y0) - a)
                                eta < f_inv_of(f, a, b, y0) - a
                                lt_add_pos(f_inv_of(f, a, b, y0) - eta, eta)
                                eta.is_positive
                                f_inv_of(f, a, b, y0) - eta < (f_inv_of(f, a, b, y0) - eta) + eta
                                (f_inv_of(f, a, b, y0) - eta) + eta = f_inv_of(f, a, b, y0)
                                f_inv_of(f, a, b, y0) - eta < f_inv_of(f, a, b, y0)
                                lt_add_right(eta, f_inv_of(f, a, b, y0) - a, a)
                                a + eta < a + (f_inv_of(f, a, b, y0) - a)
                                add_sub_cancel_shift(a, f_inv_of(f, a, b, y0))
                                a + (f_inv_of(f, a, b, y0) - a) = f_inv_of(f, a, b, y0)
                                a + eta < f_inv_of(f, a, b, y0)
                                lt_add_right(a + eta, f_inv_of(f, a, b, y0), -eta)
                                (a + eta) + -eta < f_inv_of(f, a, b, y0) + -eta
                                sub_cancels(a, eta)
                                (a + eta) + -eta = a
                                a < f_inv_of(f, a, b, y0) + -eta
                                f_inv_of(f, a, b, y0) + -eta = f_inv_of(f, a, b, y0) - eta
                                a < f_inv_of(f, a, b, y0) - eta
                                lt_imp_lte(a, f_inv_of(f, a, b, y0) - eta)
                                a <= f_inv_of(f, a, b, y0) - eta
                                lt_trans[Real](f_inv_of(f, a, b, y0) - eta,
                                    f_inv_of(f, a, b, y0), b)
                                f_inv_of(f, a, b, y0) - eta < b
                                lt_imp_lte(f_inv_of(f, a, b, y0) - eta, b)
                                f_inv_of(f, a, b, y0) - eta <= b
                                closed_interval_set_contains_of_bounds(a, b,
                                    f_inv_of(f, a, b, y0) - eta)
                                closed_interval_set(a, b).contains(f_inv_of(f, a, b, y0) - eta)
                                lt_add_pos(f_inv_of(f, a, b, y0), eta)
                                eta.is_positive
                                f_inv_of(f, a, b, y0) < f_inv_of(f, a, b, y0) + eta
                                lt_add_right(eta, b - f_inv_of(f, a, b, y0),
                                    f_inv_of(f, a, b, y0))
                                f_inv_of(f, a, b, y0) + eta < f_inv_of(f, a, b, y0) + (b - f_inv_of(f, a, b, y0))
                                add_sub_cancel_shift(f_inv_of(f, a, b, y0), b)
                                f_inv_of(f, a, b, y0) + (b - f_inv_of(f, a, b, y0)) = b
                                f_inv_of(f, a, b, y0) + eta < b
                                lt_trans[Real](a, f_inv_of(f, a, b, y0),
                                    f_inv_of(f, a, b, y0) + eta)
                                a < f_inv_of(f, a, b, y0) + eta
                                lt_imp_lte(a, f_inv_of(f, a, b, y0) + eta)
                                a <= f_inv_of(f, a, b, y0) + eta
                                lt_imp_lte(f_inv_of(f, a, b, y0) + eta, b)
                                f_inv_of(f, a, b, y0) + eta <= b
                                closed_interval_set_contains_of_bounds(a, b,
                                    f_inv_of(f, a, b, y0) + eta)
                                closed_interval_set(a, b).contains(f_inv_of(f, a, b, y0) + eta)
                                increasing_on_apply(f, a, b,
                                    f_inv_of(f, a, b, y0) - eta, f_inv_of(f, a, b, y0))
                                f(f_inv_of(f, a, b, y0) - eta) < f(f_inv_of(f, a, b, y0))
                                f(f_inv_of(f, a, b, y0)) = y0
                                f(f_inv_of(f, a, b, y0) - eta) < y0
                                increasing_on_apply(f, a, b,
                                    f_inv_of(f, a, b, y0), f_inv_of(f, a, b, y0) + eta)
                                f(f_inv_of(f, a, b, y0)) < f(f_inv_of(f, a, b, y0) + eta)
                                f(f_inv_of(f, a, b, y0)) = y0
                                y0 < f(f_inv_of(f, a, b, y0) + eta)
                                lt_imp_minus_pos(f(f_inv_of(f, a, b, y0) - eta), y0)
                                (y0 - f(f_inv_of(f, a, b, y0) - eta)).is_positive
                                lt_imp_minus_pos(y0, f(f_inv_of(f, a, b, y0) + eta))
                                (f(f_inv_of(f, a, b, y0) + eta) - y0).is_positive
                                eps_smaller_than_both(y0 - f(f_inv_of(f, a, b, y0) - eta),
                                    f(f_inv_of(f, a, b, y0) + eta) - y0)
                                let delta: Real satisfy {
                                    delta.is_positive and
                                    delta < y0 - f(f_inv_of(f, a, b, y0) - eta) and
                                    delta < f(f_inv_of(f, a, b, y0) + eta) - y0
                                }
                                forall(y1: Real) {
                                    if closed_interval_set(f(a), f(b)).contains(y1) and
                                       y1.is_close(y0, delta) {
                                        close_imp_bounds(y1, y0, delta)
                                        y1 < y0 + delta
                                        y0 - delta < y1
                                        lt_add_right(delta, f(f_inv_of(f, a, b, y0) + eta) - y0, y0)
                                        y0 + delta < y0 + (f(f_inv_of(f, a, b, y0) + eta) - y0)
                                        add_sub_cancel_shift(y0, f(f_inv_of(f, a, b, y0) + eta))
                                        y0 + (f(f_inv_of(f, a, b, y0) + eta) - y0) =
                                            f(f_inv_of(f, a, b, y0) + eta)
                                        y0 + delta < f(f_inv_of(f, a, b, y0) + eta)
                                        lt_trans[Real](y1, y0 + delta,
                                            f(f_inv_of(f, a, b, y0) + eta))
                                        y1 < f(f_inv_of(f, a, b, y0) + eta)
                                        lt_add_right(delta, y0 - f(f_inv_of(f, a, b, y0) - eta),
                                            f(f_inv_of(f, a, b, y0) - eta))
                                        f(f_inv_of(f, a, b, y0) - eta) + delta < f(f_inv_of(f, a, b, y0) - eta) +
                                            (y0 - f(f_inv_of(f, a, b, y0) - eta))
                                add_sub_cancel_shift(f(f_inv_of(f, a, b, y0) - eta), y0)
                                        f(f_inv_of(f, a, b, y0) - eta) +
                                            (y0 - f(f_inv_of(f, a, b, y0) - eta)) = y0
                                        f(f_inv_of(f, a, b, y0) - eta) + delta < y0
                                        lt_trans[Real](f(f_inv_of(f, a, b, y0) - eta) + delta,
                                            y0, y1 + delta)
                                        f(f_inv_of(f, a, b, y0) - eta) + delta < y1 + delta
                                        lt_add_converse(f(f_inv_of(f, a, b, y0) - eta), y1, delta)
                                        f(f_inv_of(f, a, b, y0) - eta) < y1
                                        lt_trans[Real](f_inv_of(f, a, b, y0) - eta,
                                            f_inv_of(f, a, b, y0), f_inv_of(f, a, b, y0) + eta)
                                        f_inv_of(f, a, b, y0) - eta < f_inv_of(f, a, b, y0) + eta
                                        f_inv_between_trap(f, a, b,
                                            f_inv_of(f, a, b, y0) - eta,
                                            f_inv_of(f, a, b, y0) + eta, y1)
                                        f_inv_of(f, a, b, y0) - eta < f_inv_of(f, a, b, y1)
                                        f_inv_of(f, a, b, y1) < f_inv_of(f, a, b, y0) + eta
                                        bounds_imp_close(f_inv_of(f, a, b, y1),
                                            f_inv_of(f, a, b, y0), eta)
                                        f_inv_of(f, a, b, y1).is_close(
                                            f_inv_of(f, a, b, y0), eta)
                                        close_and_lt_imp_close(f_inv_of(f, a, b, y1),
                                            f_inv_of(f, a, b, y0), eta, eps)
                                        eta < eps
                                        f_inv_of(f, a, b, y1).is_close(
                                            f_inv_of(f, a, b, y0), eps)
                                    }
                                }
                                continuous_on_condition(f_inv_of(f, a, b),
                                    f(a), f(b), y0, delta, eps)
                                delta.is_positive and continuous_on_condition(
                                    f_inv_of(f, a, b), f(a), f(b), y0, delta, eps)
                                exists(delta2: Real) {
                                    delta2.is_positive and continuous_on_condition(
                                        f_inv_of(f, a, b), f(a), f(b), y0, delta2, eps)
                                }
                            } else {
                                // ===== right endpoint: y0 = f(b) =====
                                not_lt_imp_gte[Real](f_inv_of(f, a, b, y0), b)
                                f_inv_of(f, a, b, y0) >= b
                                b <= f_inv_of(f, a, b, y0)
                                lte_antisymm[Real](f_inv_of(f, a, b, y0), b)
                                f_inv_of(f, a, b, y0) = b
                                lt_imp_minus_pos(a, f_inv_of(f, a, b, y0))
                                (f_inv_of(f, a, b, y0) - a).is_positive
                                eps_smaller_than_both(eps, f_inv_of(f, a, b, y0) - a)
                                let eta: Real satisfy {
                                    eta.is_positive and eta < eps and
                                    eta < f_inv_of(f, a, b, y0) - a
                                }
                                lt_add_pos(f_inv_of(f, a, b, y0) - eta, eta)
                                eta.is_positive
                                f_inv_of(f, a, b, y0) - eta < (f_inv_of(f, a, b, y0) - eta) + eta
                                (f_inv_of(f, a, b, y0) - eta) + eta = f_inv_of(f, a, b, y0)
                                f_inv_of(f, a, b, y0) - eta < f_inv_of(f, a, b, y0)
                                lt_add_right(eta, f_inv_of(f, a, b, y0) - a, a)
                                a + eta < a + (f_inv_of(f, a, b, y0) - a)
                                add_sub_cancel_shift(a, f_inv_of(f, a, b, y0))
                                a + (f_inv_of(f, a, b, y0) - a) = f_inv_of(f, a, b, y0)
                                a + eta < f_inv_of(f, a, b, y0)
                                lt_add_right(a + eta, f_inv_of(f, a, b, y0), -eta)
                                (a + eta) + -eta < f_inv_of(f, a, b, y0) + -eta
                                sub_cancels(a, eta)
                                (a + eta) + -eta = a
                                a < f_inv_of(f, a, b, y0) + -eta
                                f_inv_of(f, a, b, y0) + -eta = f_inv_of(f, a, b, y0) - eta
                                a < f_inv_of(f, a, b, y0) - eta
                                lt_imp_lte(a, f_inv_of(f, a, b, y0) - eta)
                                a <= f_inv_of(f, a, b, y0) - eta
                                lt_trans[Real](f_inv_of(f, a, b, y0) - eta,
                                    f_inv_of(f, a, b, y0), b)
                                f_inv_of(f, a, b, y0) - eta < b
                                lt_imp_lte(f_inv_of(f, a, b, y0) - eta, b)
                                f_inv_of(f, a, b, y0) - eta <= b
                                closed_interval_set_contains_of_bounds(a, b,
                                    f_inv_of(f, a, b, y0) - eta)
                                closed_interval_set(a, b).contains(f_inv_of(f, a, b, y0) - eta)
                                increasing_on_apply(f, a, b,
                                    f_inv_of(f, a, b, y0) - eta, f_inv_of(f, a, b, y0))
                                f(f_inv_of(f, a, b, y0) - eta) < f(f_inv_of(f, a, b, y0))
                                f(f_inv_of(f, a, b, y0)) = y0
                                f(f_inv_of(f, a, b, y0) - eta) < y0
                                lt_imp_minus_pos(f(f_inv_of(f, a, b, y0) - eta), y0)
                                (y0 - f(f_inv_of(f, a, b, y0) - eta)).is_positive
                                smaller_pos(y0 - f(f_inv_of(f, a, b, y0) - eta))
                                let delta: Real satisfy {
                                    delta.is_positive and
                                    delta < y0 - f(f_inv_of(f, a, b, y0) - eta)
                                }
                                forall(y1: Real) {
                                    if closed_interval_set(f(a), f(b)).contains(y1) and
                                       y1.is_close(y0, delta) {
                                        close_imp_bounds(y1, y0, delta)
                                        y0 - delta < y1
                                        lt_add_right(delta, y0 - f(f_inv_of(f, a, b, y0) - eta),
                                            f(f_inv_of(f, a, b, y0) - eta))
                                        f(f_inv_of(f, a, b, y0) - eta) + delta < f(f_inv_of(f, a, b, y0) - eta) +
                                            (y0 - f(f_inv_of(f, a, b, y0) - eta))
                                add_sub_cancel_shift(f(f_inv_of(f, a, b, y0) - eta), y0)
                                        f(f_inv_of(f, a, b, y0) - eta) +
                                            (y0 - f(f_inv_of(f, a, b, y0) - eta)) = y0
                                        f(f_inv_of(f, a, b, y0) - eta) + delta < y0
                                        lt_trans[Real](f(f_inv_of(f, a, b, y0) - eta) + delta,
                                            y0, y1 + delta)
                                        f(f_inv_of(f, a, b, y0) - eta) + delta < y1 + delta
                                        lt_add_converse(f(f_inv_of(f, a, b, y0) - eta), y1, delta)
                                        f(f_inv_of(f, a, b, y0) - eta) < y1
                                        f_inv_spec(f, a, b, y1)
                                        closed_interval_set(a, b).contains(f_inv_of(f, a, b, y1))
                                        f(f_inv_of(f, a, b, y1)) = y1
                                        f(f_inv_of(f, a, b, y0) - eta) < f(f_inv_of(f, a, b, y1))
                                        increasing_on_values_imp_args_lt(f, a, b,
                                            f_inv_of(f, a, b, y0) - eta,
                                            f_inv_of(f, a, b, y1))
                                        f_inv_of(f, a, b, y0) - eta < f_inv_of(f, a, b, y1)
                                        closed_interval_set_le_upper(a, b,
                                            f_inv_of(f, a, b, y1))
                                        f_inv_of(f, a, b, y1) <= b
                                        lt_add_pos(f_inv_of(f, a, b, y0), eta)
                                        eta.is_positive
                                        f_inv_of(f, a, b, y0) < f_inv_of(f, a, b, y0) + eta
                                        lt_of_lte_of_lt[Real](f_inv_of(f, a, b, y1),
                                            f_inv_of(f, a, b, y0), f_inv_of(f, a, b, y0) + eta)
                                        f_inv_of(f, a, b, y1) < f_inv_of(f, a, b, y0) + eta
                                        bounds_imp_close(f_inv_of(f, a, b, y1),
                                            f_inv_of(f, a, b, y0), eta)
                                        f_inv_of(f, a, b, y1).is_close(
                                            f_inv_of(f, a, b, y0), eta)
                                        close_and_lt_imp_close(f_inv_of(f, a, b, y1),
                                            f_inv_of(f, a, b, y0), eta, eps)
                                        eta < eps
                                        f_inv_of(f, a, b, y1).is_close(
                                            f_inv_of(f, a, b, y0), eps)
                                    }
                                }
                                continuous_on_condition(f_inv_of(f, a, b),
                                    f(a), f(b), y0, delta, eps)
                                delta.is_positive and continuous_on_condition(
                                    f_inv_of(f, a, b), f(a), f(b), y0, delta, eps)
                                exists(delta2: Real) {
                                    delta2.is_positive and continuous_on_condition(
                                        f_inv_of(f, a, b), f(a), f(b), y0, delta2, eps)
                                }
                            }
                        } else {
                            // ===== left endpoint: y0 = f(a) =====
                            not_lt_imp_gte[Real](a, f_inv_of(f, a, b, y0))
                            a >= f_inv_of(f, a, b, y0)
                            f_inv_of(f, a, b, y0) <= a
                            lte_antisymm[Real](a, f_inv_of(f, a, b, y0))
                            a = f_inv_of(f, a, b, y0)
                            f_inv_of(f, a, b, y0) = a
                            lt_imp_minus_pos(f_inv_of(f, a, b, y0), b)
                            (b - f_inv_of(f, a, b, y0)).is_positive
                            eps_smaller_than_both(eps, b - f_inv_of(f, a, b, y0))
                            let eta: Real satisfy {
                                eta.is_positive and eta < eps and
                                eta < b - f_inv_of(f, a, b, y0)
                            }
                            lt_add_pos(f_inv_of(f, a, b, y0), eta)
                            eta.is_positive
                            f_inv_of(f, a, b, y0) < f_inv_of(f, a, b, y0) + eta
                            lt_add_right(eta, b - f_inv_of(f, a, b, y0),
                                f_inv_of(f, a, b, y0))
                            f_inv_of(f, a, b, y0) + eta < f_inv_of(f, a, b, y0) + (b - f_inv_of(f, a, b, y0))
                                add_sub_cancel_shift(f_inv_of(f, a, b, y0), b)
                            f_inv_of(f, a, b, y0) + (b - f_inv_of(f, a, b, y0)) = b
                            f_inv_of(f, a, b, y0) + eta < b
                            f_inv_of(f, a, b, y0) = a
                            a < f_inv_of(f, a, b, y0) + eta
                            lt_imp_lte(a, f_inv_of(f, a, b, y0) + eta)
                            a <= f_inv_of(f, a, b, y0) + eta
                            lt_imp_lte(f_inv_of(f, a, b, y0) + eta, b)
                            f_inv_of(f, a, b, y0) + eta <= b
                            closed_interval_set_contains_of_bounds(a, b,
                                f_inv_of(f, a, b, y0) + eta)
                            closed_interval_set(a, b).contains(f_inv_of(f, a, b, y0) + eta)
                            increasing_on_apply(f, a, b,
                                f_inv_of(f, a, b, y0), f_inv_of(f, a, b, y0) + eta)
                            f(f_inv_of(f, a, b, y0)) < f(f_inv_of(f, a, b, y0) + eta)
                            f(f_inv_of(f, a, b, y0)) = y0
                            y0 < f(f_inv_of(f, a, b, y0) + eta)
                            lt_imp_minus_pos(y0, f(f_inv_of(f, a, b, y0) + eta))
                            (f(f_inv_of(f, a, b, y0) + eta) - y0).is_positive
                            smaller_pos(f(f_inv_of(f, a, b, y0) + eta) - y0)
                            let delta: Real satisfy {
                                delta.is_positive and
                                delta < f(f_inv_of(f, a, b, y0) + eta) - y0
                            }
                            forall(y1: Real) {
                                if closed_interval_set(f(a), f(b)).contains(y1) and
                                   y1.is_close(y0, delta) {
                                    close_imp_bounds(y1, y0, delta)
                                    y1 < y0 + delta
                                    lt_add_right(delta, f(f_inv_of(f, a, b, y0) + eta) - y0, y0)
                                    y0 + delta < y0 + (f(f_inv_of(f, a, b, y0) + eta) - y0)
                                    add_sub_cancel_shift(y0, f(f_inv_of(f, a, b, y0) + eta))
                                    y0 + (f(f_inv_of(f, a, b, y0) + eta) - y0) =
                                        f(f_inv_of(f, a, b, y0) + eta)
                                    y0 + delta < f(f_inv_of(f, a, b, y0) + eta)
                                    lt_trans[Real](y1, y0 + delta,
                                        f(f_inv_of(f, a, b, y0) + eta))
                                    y1 < f(f_inv_of(f, a, b, y0) + eta)
                                    f_inv_spec(f, a, b, y1)
                                    closed_interval_set(a, b).contains(f_inv_of(f, a, b, y1))
                                    f(f_inv_of(f, a, b, y1)) = y1
                                    f(f_inv_of(f, a, b, y1)) < f(f_inv_of(f, a, b, y0) + eta)
                                    increasing_on_values_imp_args_lt(f, a, b,
                                        f_inv_of(f, a, b, y1),
                                        f_inv_of(f, a, b, y0) + eta)
                                    f_inv_of(f, a, b, y1) < f_inv_of(f, a, b, y0) + eta
                                    closed_interval_set_lower_le(a, b,
                                        f_inv_of(f, a, b, y1))
                                    a <= f_inv_of(f, a, b, y1)
                                    f_inv_of(f, a, b, y0) = a
                                    f_inv_of(f, a, b, y0) <= f_inv_of(f, a, b, y1)
                                    lt_add_pos(f_inv_of(f, a, b, y0) - eta, eta)
                                    eta.is_positive
                                    f_inv_of(f, a, b, y0) - eta < (f_inv_of(f, a, b, y0) - eta) + eta
                                    (f_inv_of(f, a, b, y0) - eta) + eta =
                                        f_inv_of(f, a, b, y0)
                                    f_inv_of(f, a, b, y0) - eta < f_inv_of(f, a, b, y0)
                                    lt_of_lt_of_lte[Real](f_inv_of(f, a, b, y0) - eta,
                                        f_inv_of(f, a, b, y0), f_inv_of(f, a, b, y1))
                                    f_inv_of(f, a, b, y0) - eta < f_inv_of(f, a, b, y1)
                                    bounds_imp_close(f_inv_of(f, a, b, y1),
                                        f_inv_of(f, a, b, y0), eta)
                                    f_inv_of(f, a, b, y1).is_close(
                                        f_inv_of(f, a, b, y0), eta)
                                    close_and_lt_imp_close(f_inv_of(f, a, b, y1),
                                        f_inv_of(f, a, b, y0), eta, eps)
                                    eta < eps
                                    f_inv_of(f, a, b, y1).is_close(
                                        f_inv_of(f, a, b, y0), eps)
                                }
                            }
                            continuous_on_condition(f_inv_of(f, a, b),
                                f(a), f(b), y0, delta, eps)
                            delta.is_positive and continuous_on_condition(
                                f_inv_of(f, a, b), f(a), f(b), y0, delta, eps)
                            exists(delta2: Real) {
                                delta2.is_positive and continuous_on_condition(
                                    f_inv_of(f, a, b), f(a), f(b), y0, delta2, eps)
                            }
                        }
                    }
                }
            }
        }
    }
}
