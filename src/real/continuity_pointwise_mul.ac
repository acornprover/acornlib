from data.basic.function_algebra import pointwise_mul
from real.continuity_composition import continuous_at_delta, continuous_at_intro,
    continuous_intro
from real.continuity_local_bounded import continuous_at_local_abs_bound
from real.continuity_base import Real, continuous, continuous_at, continuous_condition
from real.prod_seq import mul_close_from_close
from real.real_ring import exists_small_mul_variant_2
from real.real_seq import close_and_lt_imp_close, eps_lt_half, eps_smaller_than_both

/// Pointwise multiplication preserves continuity at a point.
theorem continuous_at_pointwise_mul(f: Real -> Real, g: Real -> Real, x: Real) {
    continuous_at(f, x) and continuous_at(g, x)
    implies continuous_at(pointwise_mul[Real, Real](f, g), x)
} by {
    if continuous_at(f, x) and continuous_at(g, x) {
        continuous_at_local_abs_bound(f, x)
        let (delta_b: Real, f_bound: Real) satisfy {
            delta_b.is_positive and f_bound.is_positive and forall(y: Real) {
                y.is_close(x, delta_b) implies f(y).abs <= f_bound
            }
        }
        not g(x).abs.is_negative
        let g_bound: Real = g(x).abs + Real.1
        g(x).abs + Real.1 >= Real.0 + Real.1
        Real.0 + Real.1 = Real.1
        Real.1.is_positive
        g_bound >= Real.1
        g_bound.is_positive
        g(x).abs <= g_bound

        forall(eps: Real) {
            if eps.is_positive {
                eps_lt_half(eps)
                let eps2: Real satisfy {
                    eps2.is_positive and eps2 + eps2 < eps
                }
                exists_small_mul_variant_2(g_bound, eps2)
                let eps_a: Real satisfy {
                    eps_a.is_positive and eps_a * g_bound < eps2
                }
                exists_small_mul_variant_2(f_bound, eps2)
                let eps_b: Real satisfy {
                    eps_b.is_positive and eps_b * f_bound < eps2
                }
                continuous_at_delta(f, x, eps_a)
                let delta_f: Real satisfy {
                    delta_f.is_positive and continuous_condition(f, x, delta_f, eps_a)
                }
                continuous_at_delta(g, x, eps_b)
                let delta_g: Real satisfy {
                    delta_g.is_positive and continuous_condition(g, x, delta_g, eps_b)
                }
                eps_smaller_than_both(delta_f, delta_g)
                let delta_fg: Real satisfy {
                    delta_fg.is_positive and delta_fg < delta_f and delta_fg < delta_g
                }
                eps_smaller_than_both(delta_fg, delta_b)
                let delta: Real satisfy {
                    delta.is_positive and delta < delta_fg and delta < delta_b
                }
                delta < delta_f
                delta < delta_g
                continuous_condition(f, x, delta_f, eps_a) = forall(y: Real) {
                    y.is_close(x, delta_f) implies f(y).is_close(f(x), eps_a)
                }
                continuous_condition(g, x, delta_g, eps_b) = forall(y: Real) {
                    y.is_close(x, delta_g) implies g(y).is_close(g(x), eps_b)
                }
                forall(y: Real) {
                    if y.is_close(x, delta) {
                        close_and_lt_imp_close(y, x, delta, delta_f)
                        close_and_lt_imp_close(y, x, delta, delta_g)
                        close_and_lt_imp_close(y, x, delta, delta_b)
                        y.is_close(x, delta_f)
                        y.is_close(x, delta_g)
                        y.is_close(x, delta_b)
                        f(y).is_close(f(x), eps_a)
                        g(y).is_close(g(x), eps_b)
                        f(y).abs <= f_bound
                        mul_close_from_close(f(y), f(x), g(y), g(x), eps_a, eps_b, f_bound, g_bound)
                        (f(y) * g(y)).is_close(f(x) * g(x), f_bound * eps_b + eps_a * g_bound)
                        f_bound * eps_b = eps_b * f_bound
                        f_bound * eps_b < eps2
                        eps_a * g_bound < eps2
                        let total: Real = f_bound * eps_b + eps_a * g_bound
                        total = f_bound * eps_b + eps_a * g_bound
                        total < eps2 + eps2
                        eps2 + eps2 < eps
                        total < eps
                        f_bound * eps_b + eps_a * g_bound < eps
                        close_and_lt_imp_close(f(y) * g(y), f(x) * g(x), f_bound * eps_b + eps_a * g_bound, eps)
                        (f(y) * g(y)).is_close(f(x) * g(x), eps)
                        pointwise_mul[Real, Real](f, g, y) = f(y) * g(y)
                        pointwise_mul[Real, Real](f, g, x) = f(x) * g(x)
                        pointwise_mul[Real, Real](f, g, y).is_close(pointwise_mul[Real, Real](f, g, x), eps)
                    }
                }
                continuous_condition(pointwise_mul[Real, Real](f, g), x, delta, eps)
                delta.is_positive and continuous_condition(pointwise_mul[Real, Real](f, g), x, delta, eps)
                exists(d: Real) {
                    d.is_positive and continuous_condition(pointwise_mul[Real, Real](f, g), x, d, eps)
                }
            }
        }
        continuous_at(pointwise_mul[Real, Real](f, g), x) = forall(eps3: Real) {
            eps3.is_positive implies exists(delta3: Real) {
                delta3.is_positive and continuous_condition(pointwise_mul[Real, Real](f, g), x, delta3, eps3)
            }
        }
        if not continuous_at(pointwise_mul[Real, Real](f, g), x) {
            not forall(eps3: Real) {
                eps3.is_positive implies exists(delta3: Real) {
                    delta3.is_positive and continuous_condition(pointwise_mul[Real, Real](f, g), x, delta3, eps3)
                }
            }
            let bad_eps: Real satisfy {
                bad_eps.is_positive and forall(delta3: Real) {
                    not (delta3.is_positive and continuous_condition(pointwise_mul[Real, Real](f, g), x, delta3, bad_eps))
                }
            }
            exists(delta3: Real) {
                delta3.is_positive and continuous_condition(pointwise_mul[Real, Real](f, g), x, delta3, bad_eps)
            }
            false
        }
        continuous_at(pointwise_mul[Real, Real](f, g), x)
    }
}

/// Pointwise multiplication preserves continuous real functions.
theorem continuous_pointwise_mul(f: Real -> Real, g: Real -> Real) {
    continuous(f) and continuous(g) implies continuous(pointwise_mul[Real, Real](f, g))
} by {
    continuous(f) = forall(x: Real) {
        continuous_at(f, x)
    }
    continuous(g) = forall(x: Real) {
        continuous_at(g, x)
    }
    forall(x: Real) {
        continuous_at(f, x)
        continuous_at(g, x)
        continuous_at_pointwise_mul(f, g, x)
        continuous_at(pointwise_mul[Real, Real](f, g), x)
    }
    continuous_intro(pointwise_mul[Real, Real](f, g))
    continuous(pointwise_mul[Real, Real](f, g))
}
