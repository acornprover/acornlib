from nat import Nat, mul_suc_right, lte_mul_both
from rat import Rat
from list import partial, partial_scalar_mul
from algebra.semigroup import mul_fn
from real.real_field import Real
from real.real_ring import converges, limit, converges_to, mul_abs, mul_zero_left, mul_zero_right, mul_neg_one_left, mul_neg_right, real_mul_comm, lte_mul_nonneg_right
from real.real_seq import eq_imp_limit, eventual_eq, converges_to_imp_converges, converges_imp_converges_to, converges_to_unique
from real.real_base import abs_neg, neg_neg, abs_gte_zero, lte_add_right, lte_trans, pos_imp_eq_abs
from real.abs_conv import absolutely_converges, abs_fn, abs_fn_nonneg, absolutely_converges_imp_converges
from real.real_series import partial_suc, partial_zero, is_lower_bound, is_upper_bound, has_upper_bound_seq, is_increasing, nonneg_partial_increasing, increasing_convergent_bounded_by_limit, increasing_bounded_above_converges, const_converges, const_limit, tail, tail_imp_converges_to, mul_seq, neg_seq, limit_neg_seq, abs_pow, pow_nonneg
from real.exp import exp_term, exp_term_abs, exp_term_partial_converges, exp_term_zero_index, factorial_pos, pow_suc, zero_pow_pos, inverse_pos, abs_div
from real.derivative_rules import neg_div
from algebra.ring.ring import alternating_sign, alternating_sign_zero, alternating_sign_suc, alternating_sign_eq_neg_one_pow

numerals Real
numerals Nat

/// The nth term in the Taylor series of sine: (-1)^n x^(2n+1) / (2n+1)!.
define sin_term(x: Real, n: Nat) -> Real {
    alternating_sign[Real](n) * x.pow(Nat.2 * n + Nat.1) / Real.from_rat(Rat.from_nat((Nat.2 * n + Nat.1).factorial))
}

/// The nth term in the Taylor series of cosine: (-1)^n x^(2n) / (2n)!.
define cos_term(x: Real, n: Nat) -> Real {
    alternating_sign[Real](n) * x.pow(Nat.2 * n) / Real.from_rat(Rat.from_nat((Nat.2 * n).factorial))
}

/// The sine and cosine functions, defined by their power series.
attributes Real {
    /// The sine function, defined by its power series.
    define sin(self) -> Real {
        limit(partial(sin_term(self)))
    }

    /// The cosine function, defined by its power series.
    define cos(self) -> Real {
        limit(partial(cos_term(self)))
    }
}

// Alternating sign helpers for the trigonometric series.

/// The absolute value of one is one.
theorem abs_one {
    Real.1.abs = Real.1
} by {
    Real.1 > Real.0
    Real.1.is_positive
    pos_imp_eq_abs(Real.1)
    Real.1 = Real.1.abs
    Real.1.abs = Real.1
}

/// Doubling a successor is two more than doubling.
theorem two_mul_suc(k: Nat) {
    Nat.2 * k.suc = (Nat.2 * k).suc.suc
} by {
    mul_suc_right(Nat.2, k)
    Nat.2 * k.suc = Nat.2 + Nat.2 * k
    Nat.2 + Nat.2 * k = Nat.2 * k + Nat.2
    Nat.2 * k + Nat.2 = (Nat.2 * k).suc.suc
    Nat.2 * k.suc = (Nat.2 * k).suc.suc
}

/// The alternating sign flips at a successor.
theorem alt_abs_step(k: Nat) {
    alternating_sign[Real](k).abs = Real.1 implies alternating_sign[Real](k.suc).abs = Real.1
} by {
    if alternating_sign[Real](k).abs = Real.1 {
        alternating_sign_suc[Real](k)
        alternating_sign[Real](k.suc) = -alternating_sign[Real](k)
        alternating_sign[Real](k.suc).abs = (-alternating_sign[Real](k)).abs
        abs_neg(alternating_sign[Real](k))
        (-alternating_sign[Real](k)).abs = alternating_sign[Real](k).abs
        alternating_sign[Real](k).abs = Real.1
        alternating_sign[Real](k.suc).abs = Real.1
    }
}

/// The alternating sign is plus or minus one.
theorem alternating_sign_abs(n: Nat) {
    alternating_sign[Real](n).abs = Real.1
} by {
    define p(k: Nat) -> Bool {
        alternating_sign[Real](k).abs = Real.1
    }
    alternating_sign_zero[Real]
    alternating_sign[Real](Nat.0) = Real.1
    abs_one
    Real.1.abs = Real.1
    alternating_sign[Real](Nat.0).abs = Real.1
    p(Nat.0)
    forall(k: Nat) {
        alt_abs_step(k)
        alternating_sign[Real](k).abs = Real.1 implies alternating_sign[Real](k.suc).abs = Real.1
        if p(k) {
            p(k.suc)
        }
    }
    p(Nat.0) and forall(k: Nat) { p(k) implies p(k.suc) }
    Nat.induction(p)
}

/// The alternating sign stays one when stepping from an even index.
theorem alt_double_step(k: Nat) {
    alternating_sign[Real](Nat.2 * k) = Real.1 implies alternating_sign[Real](Nat.2 * k.suc) = Real.1
} by {
    if alternating_sign[Real](Nat.2 * k) = Real.1 {
        alternating_sign_suc[Real](Nat.2 * k)
        alternating_sign_suc[Real]((Nat.2 * k).suc)
        alternating_sign[Real]((Nat.2 * k).suc) = -alternating_sign[Real](Nat.2 * k)
        alternating_sign[Real]((Nat.2 * k).suc.suc) = -alternating_sign[Real]((Nat.2 * k).suc)
        alternating_sign[Real]((Nat.2 * k).suc.suc) = --alternating_sign[Real](Nat.2 * k)
        neg_neg(alternating_sign[Real](Nat.2 * k))
        --alternating_sign[Real](Nat.2 * k) = alternating_sign[Real](Nat.2 * k)
        two_mul_suc(k)
        Nat.2 * k.suc = (Nat.2 * k).suc.suc
        alternating_sign[Real](Nat.2 * k.suc) = alternating_sign[Real]((Nat.2 * k).suc.suc)
        alternating_sign[Real](Nat.2 * k.suc) = alternating_sign[Real](Nat.2 * k)
        alternating_sign[Real](Nat.2 * k) = Real.1
        alternating_sign[Real](Nat.2 * k.suc) = Real.1
    }
}

/// The alternating sign at an even index is one.
theorem alternating_sign_double(n: Nat) {
    alternating_sign[Real](Nat.2 * n) = Real.1
} by {
    define p(k: Nat) -> Bool {
        alternating_sign[Real](Nat.2 * k) = Real.1
    }

    alternating_sign_zero[Real]
    alternating_sign[Real](Nat.0) = Real.1
    Nat.2 * Nat.0 = Nat.0
    alternating_sign[Real](Nat.2 * Nat.0) = Real.1
    p(Nat.0)

    forall(k: Nat) {
        alt_double_step(k)
        alternating_sign[Real](Nat.2 * k) = Real.1 implies alternating_sign[Real](Nat.2 * k.suc) = Real.1
        if p(k) {
            p(k.suc)
        }
    }

    p(Nat.0) and forall(k: Nat) { p(k) implies p(k.suc) }
    Nat.induction(p)
}

/// The alternating sign flips when stepping from an odd index.
theorem alt_double_suc_step(k: Nat) {
    alternating_sign[Real](Nat.2 * k + Nat.1) = -Real.1 implies alternating_sign[Real](Nat.2 * k.suc + Nat.1) = -Real.1
} by {
    if alternating_sign[Real](Nat.2 * k + Nat.1) = -Real.1 {
        alternating_sign_suc[Real](Nat.2 * k + Nat.1)
        alternating_sign_suc[Real]((Nat.2 * k + Nat.1).suc)
        alternating_sign[Real]((Nat.2 * k + Nat.1).suc) = -alternating_sign[Real](Nat.2 * k + Nat.1)
        alternating_sign[Real]((Nat.2 * k + Nat.1).suc.suc) = -alternating_sign[Real]((Nat.2 * k + Nat.1).suc)
        alternating_sign[Real]((Nat.2 * k + Nat.1).suc.suc) = --alternating_sign[Real](Nat.2 * k + Nat.1)
        neg_neg(alternating_sign[Real](Nat.2 * k + Nat.1))
        --alternating_sign[Real](Nat.2 * k + Nat.1) = alternating_sign[Real](Nat.2 * k + Nat.1)
        Nat.2 * k.suc + Nat.1 = (Nat.2 * k + Nat.1).suc.suc
        alternating_sign[Real](Nat.2 * k.suc + Nat.1) = alternating_sign[Real]((Nat.2 * k + Nat.1).suc.suc)
        alternating_sign[Real](Nat.2 * k.suc + Nat.1) = alternating_sign[Real](Nat.2 * k + Nat.1)
        alternating_sign[Real](Nat.2 * k + Nat.1) = -Real.1
        alternating_sign[Real](Nat.2 * k.suc + Nat.1) = -Real.1
    }
}

/// The alternating sign at an odd index is negative one.
theorem alternating_sign_double_suc(n: Nat) {
    alternating_sign[Real](Nat.2 * n + Nat.1) = -Real.1
} by {
    define p(k: Nat) -> Bool {
        alternating_sign[Real](Nat.2 * k + Nat.1) = -Real.1
    }

    alternating_sign_zero[Real]
    alternating_sign[Real](Nat.0) = Real.1
    alternating_sign_suc[Real](Nat.0)
    alternating_sign[Real](Nat.1) = -alternating_sign[Real](Nat.0)
    alternating_sign[Real](Nat.1) = -Real.1
    Nat.2 * Nat.0 + Nat.1 = Nat.1
    alternating_sign[Real](Nat.2 * Nat.0 + Nat.1) = -Real.1
    p(Nat.0)

    forall(k: Nat) {
        alt_double_suc_step(k)
        alternating_sign[Real](Nat.2 * k + Nat.1) = -Real.1 implies alternating_sign[Real](Nat.2 * k.suc + Nat.1) = -Real.1
        if p(k) {
            p(k.suc)
        }
    }

    p(n)
}

/// Every successor is at least one.
theorem suc_ge_one(n: Nat) {
    Nat.1 <= n.suc
} by {
}

/// Doubling a number at least one stays at least two.
theorem two_mul_ge_one_step(k: Nat) {
    Nat.2 * k.suc >= Nat.1
} by {
    two_mul_suc(k)
    Nat.2 * k.suc = (Nat.2 * k).suc.suc
    suc_ge_one((Nat.2 * k).suc)
    Nat.1 <= (Nat.2 * k).suc.suc
    Nat.1 <= Nat.2 * k.suc
    Nat.2 * k.suc >= Nat.1
}

/// Doubling preserves being at least one.
theorem two_mul_ge_one(n: Nat) {
    n >= Nat.1 implies Nat.2 * n >= Nat.1
} by {
    define p(k: Nat) -> Bool {
        k >= Nat.1 implies Nat.2 * k >= Nat.1
    }
    not Nat.0 >= Nat.1
    p(Nat.0)
    forall(k: Nat) {
        two_mul_ge_one_step(k)
        Nat.2 * k.suc >= Nat.1
        if p(k) {
            p(k.suc)
        }
    }
    p(Nat.0) and forall(k: Nat) { p(k) implies p(k.suc) }
    Nat.induction(p)
}

// Absolute values of the trigonometric terms.

/// The absolute value of the nth sine term is |x|^(2n+1) / (2n+1)!.
theorem sin_term_abs(x: Real, n: Nat) {
    sin_term(x, n).abs = x.abs.pow(Nat.2 * n + Nat.1) / Real.from_rat(Rat.from_nat((Nat.2 * n + Nat.1).factorial))
} by {
    let f = Real.from_rat(Rat.from_nat((Nat.2 * n + Nat.1).factorial))
    factorial_pos(Nat.2 * n + Nat.1)
    f > Real.0
    f != Real.0
    not f.is_negative
    f.abs = f

    sin_term(x, n) = alternating_sign[Real](n) * x.pow(Nat.2 * n + Nat.1) / f
    sin_term(x, n).abs = (alternating_sign[Real](n) * x.pow(Nat.2 * n + Nat.1) / f).abs
    abs_div(alternating_sign[Real](n) * x.pow(Nat.2 * n + Nat.1), f)
    (alternating_sign[Real](n) * x.pow(Nat.2 * n + Nat.1) / f).abs =
        (alternating_sign[Real](n) * x.pow(Nat.2 * n + Nat.1)).abs / f.abs
    mul_abs(alternating_sign[Real](n), x.pow(Nat.2 * n + Nat.1))
    (alternating_sign[Real](n) * x.pow(Nat.2 * n + Nat.1)).abs =
        alternating_sign[Real](n).abs * x.pow(Nat.2 * n + Nat.1).abs
    alternating_sign_abs(n)
    alternating_sign[Real](n).abs = Real.1
    abs_pow(x, Nat.2 * n + Nat.1)
    x.pow(Nat.2 * n + Nat.1).abs = x.abs.pow(Nat.2 * n + Nat.1)
    alternating_sign[Real](n).abs * x.pow(Nat.2 * n + Nat.1).abs =
        Real.1 * x.abs.pow(Nat.2 * n + Nat.1)
    Real.1 * x.abs.pow(Nat.2 * n + Nat.1) = x.abs.pow(Nat.2 * n + Nat.1)
    (alternating_sign[Real](n) * x.pow(Nat.2 * n + Nat.1)).abs = x.abs.pow(Nat.2 * n + Nat.1)
    (alternating_sign[Real](n) * x.pow(Nat.2 * n + Nat.1) / f).abs =
        x.abs.pow(Nat.2 * n + Nat.1) / f.abs
    x.abs.pow(Nat.2 * n + Nat.1) / f.abs = x.abs.pow(Nat.2 * n + Nat.1) / f
    sin_term(x, n).abs = x.abs.pow(Nat.2 * n + Nat.1) / f
}

/// The absolute value of the nth sine term is the odd exponential term.
theorem sin_term_abs_exp(x: Real, n: Nat) {
    sin_term(x, n).abs = exp_term(x.abs, Nat.2 * n + Nat.1)
} by {
    sin_term_abs(x, n)
    sin_term(x, n).abs = x.abs.pow(Nat.2 * n + Nat.1) / Real.from_rat(Rat.from_nat((Nat.2 * n + Nat.1).factorial))
    exp_term(x.abs, Nat.2 * n + Nat.1) = x.abs.pow(Nat.2 * n + Nat.1) / Real.from_rat(Rat.from_nat((Nat.2 * n + Nat.1).factorial))
    sin_term(x, n).abs = exp_term(x.abs, Nat.2 * n + Nat.1)
}

/// The absolute value of the nth cosine term is |x|^(2n) / (2n)!.
theorem cos_term_abs(x: Real, n: Nat) {
    cos_term(x, n).abs = x.abs.pow(Nat.2 * n) / Real.from_rat(Rat.from_nat((Nat.2 * n).factorial))
} by {
    let f = Real.from_rat(Rat.from_nat((Nat.2 * n).factorial))
    factorial_pos(Nat.2 * n)
    f > Real.0
    f != Real.0
    not f.is_negative
    f.abs = f

    cos_term(x, n) = alternating_sign[Real](n) * x.pow(Nat.2 * n) / f
    cos_term(x, n).abs = (alternating_sign[Real](n) * x.pow(Nat.2 * n) / f).abs
    abs_div(alternating_sign[Real](n) * x.pow(Nat.2 * n), f)
    (alternating_sign[Real](n) * x.pow(Nat.2 * n) / f).abs =
        (alternating_sign[Real](n) * x.pow(Nat.2 * n)).abs / f.abs
    mul_abs(alternating_sign[Real](n), x.pow(Nat.2 * n))
    (alternating_sign[Real](n) * x.pow(Nat.2 * n)).abs =
        alternating_sign[Real](n).abs * x.pow(Nat.2 * n).abs
    alternating_sign_abs(n)
    alternating_sign[Real](n).abs = Real.1
    abs_pow(x, Nat.2 * n)
    x.pow(Nat.2 * n).abs = x.abs.pow(Nat.2 * n)
    alternating_sign[Real](n).abs * x.pow(Nat.2 * n).abs =
        Real.1 * x.abs.pow(Nat.2 * n)
    Real.1 * x.abs.pow(Nat.2 * n) = x.abs.pow(Nat.2 * n)
    (alternating_sign[Real](n) * x.pow(Nat.2 * n)).abs = x.abs.pow(Nat.2 * n)
    (alternating_sign[Real](n) * x.pow(Nat.2 * n) / f).abs =
        x.abs.pow(Nat.2 * n) / f.abs
    x.abs.pow(Nat.2 * n) / f.abs = x.abs.pow(Nat.2 * n) / f
    cos_term(x, n).abs = x.abs.pow(Nat.2 * n) / f
}

/// The absolute value of the nth cosine term is the even exponential term.
theorem cos_term_abs_exp(x: Real, n: Nat) {
    cos_term(x, n).abs = exp_term(x.abs, Nat.2 * n)
} by {
    cos_term_abs(x, n)
    cos_term(x, n).abs = x.abs.pow(Nat.2 * n) / Real.from_rat(Rat.from_nat((Nat.2 * n).factorial))
    exp_term(x.abs, Nat.2 * n) = x.abs.pow(Nat.2 * n) / Real.from_rat(Rat.from_nat((Nat.2 * n).factorial))
    cos_term(x, n).abs = exp_term(x.abs, Nat.2 * n)
}

/// Exponential terms are nonnegative for nonnegative bases.
theorem exp_term_nonneg(x: Real, n: Nat) {
    x >= Real.0 implies exp_term(x, n) >= Real.0
} by {
    if x >= Real.0 {
        pow_nonneg(x, n)
        Real.0 <= x.pow(n)
        let f = Real.from_rat(Rat.from_nat(n.factorial))
        factorial_pos(n)
        f > Real.0
        f != Real.0
        f.is_positive
        inverse_pos(f)
        f.inverse.is_positive
        Real.0 <= f.inverse
        x.pow(n) >= Real.0
        not f.inverse.is_negative
        lte_mul_nonneg_right(Real.0, x.pow(n), f.inverse)
        Real.0 * f.inverse <= x.pow(n) * f.inverse
        Real.0 * f.inverse = Real.0
        Real.0 <= x.pow(n) * f.inverse
        exp_term(x, n) = x.pow(n) / f
        x.pow(n) / f = x.pow(n) * f.inverse
        exp_term(x, n) = x.pow(n) * f.inverse
        exp_term(x, n) >= Real.0
    }
}

// Bounds of the partial sums of absolute trigonometric terms.

/// The first partial sum of the exponential series is nonnegative.
theorem sin_bound_base(x: Real) {
    Real.0 <= partial(exp_term(x.abs), Nat.1)
} by {
    exp_term_zero_index(x.abs)
    exp_term(x.abs, Nat.0) = Real.1
    partial(exp_term(x.abs), Nat.1) = exp_term(x.abs, Nat.0)
    partial(exp_term(x.abs), Nat.1) = Real.1
    Real.1 > Real.0
    Real.0 <= Real.1
    Real.0 <= partial(exp_term(x.abs), Nat.1)
}

/// The sine bound induction step: adding one term keeps the bound.
theorem sin_partial_bound_step(x: Real, k: Nat) {
    partial(abs_fn(sin_term(x)), k) <= partial(exp_term(x.abs), Nat.2 * k + Nat.1)
    implies
    partial(abs_fn(sin_term(x)), k.suc) <= partial(exp_term(x.abs), Nat.2 * k.suc + Nat.1)
} by {
    if partial(abs_fn(sin_term(x)), k) <= partial(exp_term(x.abs), Nat.2 * k + Nat.1) {
        partial_suc(abs_fn(sin_term(x)), k)
        partial(abs_fn(sin_term(x)), k.suc) = partial(abs_fn(sin_term(x)), k) + abs_fn(sin_term(x))(k)
        abs_fn(sin_term(x))(k) = sin_term(x, k).abs
        sin_term_abs_exp(x, k)
        sin_term(x, k).abs = exp_term(x.abs, Nat.2 * k + Nat.1)
        abs_fn(sin_term(x))(k) = exp_term(x.abs, Nat.2 * k + Nat.1)
        partial(abs_fn(sin_term(x)), k.suc) = partial(abs_fn(sin_term(x)), k) + exp_term(x.abs, Nat.2 * k + Nat.1)
        lte_add_right(partial(abs_fn(sin_term(x)), k), partial(exp_term(x.abs), Nat.2 * k + Nat.1), exp_term(x.abs, Nat.2 * k + Nat.1))
        partial(abs_fn(sin_term(x)), k) + exp_term(x.abs, Nat.2 * k + Nat.1) <= partial(exp_term(x.abs), Nat.2 * k + Nat.1) + exp_term(x.abs, Nat.2 * k + Nat.1)
        partial_suc(exp_term(x.abs), Nat.2 * k + Nat.1)
        partial(exp_term(x.abs), (Nat.2 * k + Nat.1).suc) = partial(exp_term(x.abs), Nat.2 * k + Nat.1) + exp_term(x.abs, Nat.2 * k + Nat.1)
        partial(abs_fn(sin_term(x)), k.suc) <= partial(exp_term(x.abs), (Nat.2 * k + Nat.1).suc)
        abs_gte_zero(x)
        Real.0 <= x.abs
        exp_term_nonneg(x.abs, (Nat.2 * k + Nat.1).suc)
        exp_term(x.abs, (Nat.2 * k + Nat.1).suc) >= Real.0
        lte_add_right(Real.0, exp_term(x.abs, (Nat.2 * k + Nat.1).suc), partial(exp_term(x.abs), (Nat.2 * k + Nat.1).suc))
        Real.0 + partial(exp_term(x.abs), (Nat.2 * k + Nat.1).suc) <= exp_term(x.abs, (Nat.2 * k + Nat.1).suc) + partial(exp_term(x.abs), (Nat.2 * k + Nat.1).suc)
        Real.0 + partial(exp_term(x.abs), (Nat.2 * k + Nat.1).suc) = partial(exp_term(x.abs), (Nat.2 * k + Nat.1).suc)
        exp_term(x.abs, (Nat.2 * k + Nat.1).suc) + partial(exp_term(x.abs), (Nat.2 * k + Nat.1).suc) = partial(exp_term(x.abs), (Nat.2 * k + Nat.1).suc) + exp_term(x.abs, (Nat.2 * k + Nat.1).suc)
        partial(exp_term(x.abs), (Nat.2 * k + Nat.1).suc) <= partial(exp_term(x.abs), (Nat.2 * k + Nat.1).suc) + exp_term(x.abs, (Nat.2 * k + Nat.1).suc)
        partial_suc(exp_term(x.abs), (Nat.2 * k + Nat.1).suc)
        partial(exp_term(x.abs), (Nat.2 * k + Nat.1).suc.suc) = partial(exp_term(x.abs), (Nat.2 * k + Nat.1).suc) + exp_term(x.abs, (Nat.2 * k + Nat.1).suc)
        lte_trans(partial(abs_fn(sin_term(x)), k.suc), partial(exp_term(x.abs), (Nat.2 * k + Nat.1).suc), partial(exp_term(x.abs), (Nat.2 * k + Nat.1).suc.suc))
        partial(abs_fn(sin_term(x)), k.suc) <= partial(exp_term(x.abs), (Nat.2 * k + Nat.1).suc.suc)
        Nat.2 * k.suc + Nat.1 = (Nat.2 * k + Nat.1).suc.suc
        partial(exp_term(x.abs), (Nat.2 * k + Nat.1).suc.suc) = partial(exp_term(x.abs), Nat.2 * k.suc + Nat.1)
        partial(abs_fn(sin_term(x)), k.suc) <= partial(exp_term(x.abs), Nat.2 * k.suc + Nat.1)
    }
}

/// The partial sums of the absolute sine terms are bounded by the odd-indexed
/// partial sums of the exponential series for |x|.
theorem sin_partial_bound(x: Real, n: Nat) {
    partial(abs_fn(sin_term(x)), n) <= partial(exp_term(x.abs), Nat.2 * n + Nat.1)
} by {
    define p(k: Nat) -> Bool {
        partial(abs_fn(sin_term(x)), k) <= partial(exp_term(x.abs), Nat.2 * k + Nat.1)
    }

    partial_zero(abs_fn(sin_term(x)))
    partial(abs_fn(sin_term(x)), Nat.0) = Real.0
    sin_bound_base(x)
    Real.0 <= partial(exp_term(x.abs), Nat.1)
    partial(abs_fn(sin_term(x)), Nat.0) <= partial(exp_term(x.abs), Nat.1)
    Nat.2 * Nat.0 + Nat.1 = Nat.1
    partial(abs_fn(sin_term(x)), Nat.0) <= partial(exp_term(x.abs), Nat.2 * Nat.0 + Nat.1)
    p(Nat.0)

    forall(k: Nat) {
        sin_partial_bound_step(x, k)
        partial(abs_fn(sin_term(x)), k) <= partial(exp_term(x.abs), Nat.2 * k + Nat.1) implies partial(abs_fn(sin_term(x)), k.suc) <= partial(exp_term(x.abs), Nat.2 * k.suc + Nat.1)
        if p(k) {
            p(k.suc)
        }
    }

    p(n)
}

/// The cosine bound induction step: adding one term keeps the bound.
theorem cos_partial_bound_step(x: Real, k: Nat) {
    partial(abs_fn(cos_term(x)), k) <= partial(exp_term(x.abs), Nat.2 * k)
    implies
    partial(abs_fn(cos_term(x)), k.suc) <= partial(exp_term(x.abs), Nat.2 * k.suc)
} by {
    if partial(abs_fn(cos_term(x)), k) <= partial(exp_term(x.abs), Nat.2 * k) {
        partial_suc(abs_fn(cos_term(x)), k)
        partial(abs_fn(cos_term(x)), k.suc) = partial(abs_fn(cos_term(x)), k) + abs_fn(cos_term(x))(k)
        abs_fn(cos_term(x))(k) = cos_term(x, k).abs
        cos_term_abs_exp(x, k)
        cos_term(x, k).abs = exp_term(x.abs, Nat.2 * k)
        abs_fn(cos_term(x))(k) = exp_term(x.abs, Nat.2 * k)
        partial(abs_fn(cos_term(x)), k.suc) = partial(abs_fn(cos_term(x)), k) + exp_term(x.abs, Nat.2 * k)
        lte_add_right(partial(abs_fn(cos_term(x)), k), partial(exp_term(x.abs), Nat.2 * k), exp_term(x.abs, Nat.2 * k))
        partial(abs_fn(cos_term(x)), k) + exp_term(x.abs, Nat.2 * k) <= partial(exp_term(x.abs), Nat.2 * k) + exp_term(x.abs, Nat.2 * k)
        partial_suc(exp_term(x.abs), Nat.2 * k)
        partial(exp_term(x.abs), (Nat.2 * k).suc) = partial(exp_term(x.abs), Nat.2 * k) + exp_term(x.abs, Nat.2 * k)
        partial(abs_fn(cos_term(x)), k.suc) <= partial(exp_term(x.abs), (Nat.2 * k).suc)
        abs_gte_zero(x)
        Real.0 <= x.abs
        exp_term_nonneg(x.abs, (Nat.2 * k).suc)
        exp_term(x.abs, (Nat.2 * k).suc) >= Real.0
        lte_add_right(Real.0, exp_term(x.abs, (Nat.2 * k).suc), partial(exp_term(x.abs), (Nat.2 * k).suc))
        Real.0 + partial(exp_term(x.abs), (Nat.2 * k).suc) <= exp_term(x.abs, (Nat.2 * k).suc) + partial(exp_term(x.abs), (Nat.2 * k).suc)
        Real.0 + partial(exp_term(x.abs), (Nat.2 * k).suc) = partial(exp_term(x.abs), (Nat.2 * k).suc)
        exp_term(x.abs, (Nat.2 * k).suc) + partial(exp_term(x.abs), (Nat.2 * k).suc) = partial(exp_term(x.abs), (Nat.2 * k).suc) + exp_term(x.abs, (Nat.2 * k).suc)
        partial(exp_term(x.abs), (Nat.2 * k).suc) <= partial(exp_term(x.abs), (Nat.2 * k).suc) + exp_term(x.abs, (Nat.2 * k).suc)
        partial_suc(exp_term(x.abs), (Nat.2 * k).suc)
        partial(exp_term(x.abs), (Nat.2 * k).suc.suc) = partial(exp_term(x.abs), (Nat.2 * k).suc) + exp_term(x.abs, (Nat.2 * k).suc)
        lte_trans(partial(abs_fn(cos_term(x)), k.suc), partial(exp_term(x.abs), (Nat.2 * k).suc), partial(exp_term(x.abs), (Nat.2 * k).suc.suc))
        partial(abs_fn(cos_term(x)), k.suc) <= partial(exp_term(x.abs), (Nat.2 * k).suc.suc)
        two_mul_suc(k)
        Nat.2 * k.suc = (Nat.2 * k).suc.suc
        partial(exp_term(x.abs), (Nat.2 * k).suc.suc) = partial(exp_term(x.abs), Nat.2 * k.suc)
        partial(abs_fn(cos_term(x)), k.suc) <= partial(exp_term(x.abs), Nat.2 * k.suc)
    }
}

/// The partial sums of the absolute cosine terms are bounded by the even-indexed
/// partial sums of the exponential series for |x|.
theorem cos_partial_bound(x: Real, n: Nat) {
    partial(abs_fn(cos_term(x)), n) <= partial(exp_term(x.abs), Nat.2 * n)
} by {
    define p(k: Nat) -> Bool {
        partial(abs_fn(cos_term(x)), k) <= partial(exp_term(x.abs), Nat.2 * k)
    }

    partial_zero(abs_fn(cos_term(x)))
    partial(abs_fn(cos_term(x)), Nat.0) = Real.0
    partial_zero(exp_term(x.abs))
    partial(exp_term(x.abs), Nat.0) = Real.0
    Real.0 <= Real.0
    partial(abs_fn(cos_term(x)), Nat.0) <= partial(exp_term(x.abs), Nat.0)
    Nat.2 * Nat.0 = Nat.0
    partial(abs_fn(cos_term(x)), Nat.0) <= partial(exp_term(x.abs), Nat.2 * Nat.0)
    p(Nat.0)

    forall(k: Nat) {
        cos_partial_bound_step(x, k)
        partial(abs_fn(cos_term(x)), k) <= partial(exp_term(x.abs), Nat.2 * k) implies partial(abs_fn(cos_term(x)), k.suc) <= partial(exp_term(x.abs), Nat.2 * k.suc)
        if p(k) {
            p(k.suc)
        }
    }

    p(n)
}

// Absolute convergence of the trigonometric series.

/// The sine series converges absolutely for every real number.
theorem sin_term_abs_converges(x: Real) {
    absolutely_converges(sin_term(x))
} by {
    // The absolute terms are nonnegative, so partial sums are increasing.
    forall(n: Nat) {
        abs_fn_nonneg(sin_term(x), n)
        abs_fn(sin_term(x))(n) >= Real.0
        Real.0 <= abs_fn(sin_term(x))(n)
    }
    is_lower_bound(abs_fn(sin_term(x)), Real.0)
    nonneg_partial_increasing(abs_fn(sin_term(x)))
    is_increasing(partial(abs_fn(sin_term(x))))

    // The exponential series for |x| converges.
    forall(n: Nat) {
        abs_gte_zero(x)
        Real.0 <= x.abs
        exp_term_nonneg(x.abs, n)
        exp_term(x.abs, n) >= Real.0
        Real.0 <= exp_term(x.abs, n)
    }
    is_lower_bound(exp_term(x.abs), Real.0)
    nonneg_partial_increasing(exp_term(x.abs))
    is_increasing(partial(exp_term(x.abs)))
    exp_term_partial_converges(x.abs)
    converges(partial(exp_term(x.abs)))
    increasing_convergent_bounded_by_limit(partial(exp_term(x.abs)))
    is_upper_bound(partial(exp_term(x.abs)), limit(partial(exp_term(x.abs))))

    // The sine partial sums are bounded by the exponential partial sums.
    forall(n: Nat) {
        sin_partial_bound(x, n)
        partial(abs_fn(sin_term(x)), n) <= partial(exp_term(x.abs), Nat.2 * n + Nat.1)
        partial(exp_term(x.abs), Nat.2 * n + Nat.1) <= limit(partial(exp_term(x.abs)))
        lte_trans(partial(abs_fn(sin_term(x)), n), partial(exp_term(x.abs), Nat.2 * n + Nat.1), limit(partial(exp_term(x.abs))))
        partial(abs_fn(sin_term(x)), n) <= limit(partial(exp_term(x.abs)))
    }
    is_upper_bound(partial(abs_fn(sin_term(x))), limit(partial(exp_term(x.abs))))
    has_upper_bound_seq(partial(abs_fn(sin_term(x))))

    // An increasing sequence with an upper bound converges.
    increasing_bounded_above_converges(partial(abs_fn(sin_term(x))))
    converges(partial(abs_fn(sin_term(x))))
    absolutely_converges(sin_term(x))
}

/// The cosine series converges absolutely for every real number.
theorem cos_term_abs_converges(x: Real) {
    absolutely_converges(cos_term(x))
} by {
    // The absolute terms are nonnegative, so partial sums are increasing.
    forall(n: Nat) {
        abs_fn_nonneg(cos_term(x), n)
        abs_fn(cos_term(x))(n) >= Real.0
        Real.0 <= abs_fn(cos_term(x))(n)
    }
    is_lower_bound(abs_fn(cos_term(x)), Real.0)
    nonneg_partial_increasing(abs_fn(cos_term(x)))
    is_increasing(partial(abs_fn(cos_term(x))))

    // The exponential series for |x| converges.
    forall(n: Nat) {
        abs_gte_zero(x)
        Real.0 <= x.abs
        exp_term_nonneg(x.abs, n)
        exp_term(x.abs, n) >= Real.0
        Real.0 <= exp_term(x.abs, n)
    }
    is_lower_bound(exp_term(x.abs), Real.0)
    nonneg_partial_increasing(exp_term(x.abs))
    is_increasing(partial(exp_term(x.abs)))
    exp_term_partial_converges(x.abs)
    converges(partial(exp_term(x.abs)))
    increasing_convergent_bounded_by_limit(partial(exp_term(x.abs)))
    is_upper_bound(partial(exp_term(x.abs)), limit(partial(exp_term(x.abs))))

    // The cosine partial sums are bounded by the exponential partial sums.
    forall(n: Nat) {
        cos_partial_bound(x, n)
        partial(abs_fn(cos_term(x)), n) <= partial(exp_term(x.abs), Nat.2 * n)
        partial(exp_term(x.abs), Nat.2 * n) <= limit(partial(exp_term(x.abs)))
        lte_trans(partial(abs_fn(cos_term(x)), n), partial(exp_term(x.abs), Nat.2 * n), limit(partial(exp_term(x.abs))))
        partial(abs_fn(cos_term(x)), n) <= limit(partial(exp_term(x.abs)))
    }
    is_upper_bound(partial(abs_fn(cos_term(x))), limit(partial(exp_term(x.abs))))
    has_upper_bound_seq(partial(abs_fn(cos_term(x))))

    // An increasing sequence with an upper bound converges.
    increasing_bounded_above_converges(partial(abs_fn(cos_term(x))))
    converges(partial(abs_fn(cos_term(x))))
    absolutely_converges(cos_term(x))
}

// Basic values.

/// Every sine term at zero vanishes.
theorem sin_term_zero(n: Nat) {
    sin_term(Real.0, n) = Real.0
} by {
    pow_suc(Real.0, Nat.2 * n)
    Real.0.pow((Nat.2 * n).suc) = Real.0 * Real.0.pow(Nat.2 * n)
    Real.0 * Real.0.pow(Nat.2 * n) = Real.0
    Nat.2 * n + Nat.1 = (Nat.2 * n).suc
    Real.0.pow(Nat.2 * n + Nat.1) = Real.0
    alternating_sign[Real](n) * Real.0.pow(Nat.2 * n + Nat.1) = Real.0
    mul_zero_right(alternating_sign[Real](n))
    alternating_sign[Real](n) * Real.0 = Real.0
    let f = Real.from_rat(Rat.from_nat((Nat.2 * n + Nat.1).factorial))
    Real.0 / f = Real.0 * f.inverse
    mul_zero_left(f.inverse)
    Real.0 * f.inverse = Real.0
    Real.0 / f = Real.0
    sin_term(Real.0, n) = alternating_sign[Real](n) * Real.0.pow(Nat.2 * n + Nat.1) / Real.from_rat(Rat.from_nat((Nat.2 * n + Nat.1).factorial))
    alternating_sign[Real](n) * Real.0.pow(Nat.2 * n + Nat.1) / Real.from_rat(Rat.from_nat((Nat.2 * n + Nat.1).factorial)) = Real.0
    sin_term(Real.0, n) = Real.0
}

/// The sine partial sum induction step.
theorem sin_partial_zero_step(k: Nat) {
    partial(sin_term(Real.0), k) = Real.0 implies partial(sin_term(Real.0), k.suc) = Real.0
} by {
    if partial(sin_term(Real.0), k) = Real.0 {
        partial_suc(sin_term(Real.0), k)
        partial(sin_term(Real.0), k.suc) = partial(sin_term(Real.0), k) + sin_term(Real.0, k)
        sin_term_zero(k)
        sin_term(Real.0, k) = Real.0
        partial(sin_term(Real.0), k) = Real.0
        partial(sin_term(Real.0), k.suc) = Real.0 + Real.0
        Real.0 + Real.0 = Real.0
        partial(sin_term(Real.0), k.suc) = Real.0
    }
}

/// Every partial sum of the sine series at zero is zero.
theorem sin_partial_zero(n: Nat) {
    partial(sin_term(Real.0), n) = Real.0
} by {
    define p(k: Nat) -> Bool {
        partial(sin_term(Real.0), k) = Real.0
    }

    partial_zero(sin_term(Real.0))
    partial(sin_term(Real.0), Nat.0) = Real.0
    p(Nat.0)

    forall(k: Nat) {
        sin_partial_zero_step(k)
        partial(sin_term(Real.0), k) = Real.0 implies partial(sin_term(Real.0), k.suc) = Real.0
        if p(k) {
            p(k.suc)
        }
    }

    p(Nat.0) and forall(k: Nat) { p(k) implies p(k.suc) }
    Nat.induction(p)
}

/// The sine of zero is zero.
theorem sin_zero {
    (Real.0).sin = Real.0
} by {
    forall(i: Nat) {
        if Nat.0 <= i {
            sin_partial_zero(i)
            partial(sin_term(Real.0), i) = Real.0
        }
    }
    eventual_eq(partial(sin_term(Real.0)), Real.0)
    eq_imp_limit(partial(sin_term(Real.0)), Real.0)
    limit(partial(sin_term(Real.0))) = Real.0
    (Real.0).sin = limit(partial(sin_term(Real.0)))
    (Real.0).sin = Real.0
}

/// Every cosine term at zero beyond the zeroth vanishes.
theorem cos_term_zero(n: Nat) {
    n >= Nat.1 implies cos_term(Real.0, n) = Real.0
} by {
    if n >= Nat.1 {
        two_mul_ge_one(n)
        Nat.2 * n >= Nat.1
        zero_pow_pos(Nat.2 * n)
        Real.0.pow(Nat.2 * n) = Real.0
        alternating_sign[Real](n) * Real.0.pow(Nat.2 * n) = Real.0
        mul_zero_right(alternating_sign[Real](n))
        alternating_sign[Real](n) * Real.0 = Real.0
        let f = Real.from_rat(Rat.from_nat((Nat.2 * n).factorial))
        Real.0 / f = Real.0 * f.inverse
        mul_zero_left(f.inverse)
        Real.0 * f.inverse = Real.0
        Real.0 / f = Real.0
        cos_term(Real.0, n) = alternating_sign[Real](n) * Real.0.pow(Nat.2 * n) / Real.from_rat(Rat.from_nat((Nat.2 * n).factorial))
        alternating_sign[Real](n) * Real.0.pow(Nat.2 * n) / Real.from_rat(Rat.from_nat((Nat.2 * n).factorial)) = Real.0
        cos_term(Real.0, n) = Real.0
    }
}

/// The cosine partial sum induction step.
theorem cos_zero_partial_step(k: Nat) {
    partial(cos_term(Real.0), k.suc) = Real.1 implies partial(cos_term(Real.0), k.suc.suc) = Real.1
} by {
    if partial(cos_term(Real.0), k.suc) = Real.1 {
        partial_suc(cos_term(Real.0), k.suc)
        partial(cos_term(Real.0), k.suc.suc) = partial(cos_term(Real.0), k.suc) + cos_term(Real.0, k.suc)
        suc_ge_one(k)
        Nat.1 <= k.suc
        cos_term_zero(k.suc)
        cos_term(Real.0, k.suc) = Real.0
        partial(cos_term(Real.0), k.suc.suc) = Real.1 + Real.0
        Real.1 + Real.0 = Real.1
        partial(cos_term(Real.0), k.suc.suc) = Real.1
    }
}

/// Every partial sum of the cosine series at zero beyond the first is one.
theorem cos_zero_partial(k: Nat) {
    partial(cos_term(Real.0), k.suc) = Real.1
} by {
    define p(x: Nat) -> Bool {
        partial(cos_term(Real.0), x.suc) = Real.1
    }

    // Base case: partial at 1 is the zeroth term, which is 1.
    partial(cos_term(Real.0), Nat.1) = cos_term(Real.0, Nat.0)
    cos_term(Real.0, Nat.0) = alternating_sign[Real](Nat.0) * Real.0.pow(Nat.0) / Real.from_rat(Rat.from_nat(Nat.0.factorial))
    alternating_sign_zero[Real]
    alternating_sign[Real](Nat.0) = Real.1
    Real.0.pow(Nat.0) = Real.1
    Nat.0.factorial = Nat.1
    Real.from_rat(Rat.from_nat(Nat.1)) = Real.1
    Real.1 * Real.1 = Real.1
    Real.1 / Real.1 = Real.1
    cos_term(Real.0, Nat.0) = Real.1
    partial(cos_term(Real.0), Nat.1) = Real.1
    p(Nat.0)

    forall(x: Nat) {
        cos_zero_partial_step(x)
        partial(cos_term(Real.0), x.suc) = Real.1 implies partial(cos_term(Real.0), x.suc.suc) = Real.1
        if p(x) {
            p(x.suc)
        }
    }

    p(Nat.0) and forall(x: Nat) { p(x) implies p(x.suc) }
    Nat.induction(p)
}

/// The cosine of zero is one.
theorem cos_zero {
    (Real.0).cos = Real.1
} by {
    forall(i: Nat) {
        cos_zero_partial(i)
        partial(cos_term(Real.0), i.suc) = Real.1
        tail(partial(cos_term(Real.0)), Nat.1)(i) = partial(cos_term(Real.0))(Nat.1 + i)
        partial(cos_term(Real.0))(Nat.1 + i) = partial(cos_term(Real.0), Nat.1 + i)
        Nat.1 + i = i.suc
        partial(cos_term(Real.0), Nat.1 + i) = Real.1
        tail(partial(cos_term(Real.0)), Nat.1)(i) = Real.1
        constant[Nat, Real](Real.1, i) = Real.1
        tail(partial(cos_term(Real.0)), Nat.1)(i) = constant[Nat, Real](Real.1, i)
    }
    tail(partial(cos_term(Real.0)), Nat.1) = constant[Nat, Real](Real.1)
    const_converges(Real.1)
    converges(constant[Nat, Real](Real.1))
    tail_imp_converges_to(partial(cos_term(Real.0)), Nat.1)
    converges(tail(partial(cos_term(Real.0)), Nat.1))
    converges_to(partial(cos_term(Real.0)), limit(tail(partial(cos_term(Real.0)), Nat.1)))
    limit(tail(partial(cos_term(Real.0)), Nat.1)) = limit(constant[Nat, Real](Real.1))
    const_limit(Real.1)
    limit(constant[Nat, Real](Real.1)) = Real.1
    limit(tail(partial(cos_term(Real.0)), Nat.1)) = Real.1
    converges_to(partial(cos_term(Real.0)), Real.1)
    converges_to_imp_converges(partial(cos_term(Real.0)), Real.1)
    converges(partial(cos_term(Real.0)))
    converges_imp_converges_to(partial(cos_term(Real.0)))
    converges_to(partial(cos_term(Real.0)), limit(partial(cos_term(Real.0))))
    converges_to_unique(partial(cos_term(Real.0)), Real.1, limit(partial(cos_term(Real.0))))
    Real.1 = limit(partial(cos_term(Real.0)))
    limit(partial(cos_term(Real.0))) = Real.1
    (Real.0).cos = limit(partial(cos_term(Real.0)))
    (Real.0).cos = Real.1
}

// Parity.

/// Powers distribute over multiplication.
theorem real_pow_mul_distrib_step(x: Real, y: Real, k: Nat) {
    (x * y).pow(k) = x.pow(k) * y.pow(k)
    implies
    (x * y).pow(k.suc) = x.pow(k.suc) * y.pow(k.suc)
} by {
    if (x * y).pow(k) = x.pow(k) * y.pow(k) {
        pow_suc(x * y, k)
        (x * y).pow(k.suc) = (x * y) * (x * y).pow(k)
        (x * y).pow(k) = x.pow(k) * y.pow(k)
        (x * y).pow(k.suc) = (x * y) * (x.pow(k) * y.pow(k))
        (x * y) * (x.pow(k) * y.pow(k)) = x * y * x.pow(k) * y.pow(k)
        real_mul_comm(y, x.pow(k))
        y * x.pow(k) = x.pow(k) * y
        x * (y * x.pow(k)) * y.pow(k) = x * (x.pow(k) * y) * y.pow(k)
        x * y * x.pow(k) * y.pow(k) = x * (y * x.pow(k)) * y.pow(k)
        x * (x.pow(k) * y) * y.pow(k) = x * x.pow(k) * y * y.pow(k)
        x * y * x.pow(k) * y.pow(k) = x * x.pow(k) * y * y.pow(k)
        pow_suc(x, k)
        pow_suc(y, k)
        x.pow(k.suc) = x * x.pow(k)
        y.pow(k.suc) = y * y.pow(k)
        x * x.pow(k) * y * y.pow(k) = x.pow(k.suc) * y.pow(k.suc)
        (x * y).pow(k.suc) = x.pow(k.suc) * y.pow(k.suc)
    }
}

/// Powers distribute over multiplication.
theorem real_pow_mul_distrib(x: Real, y: Real, n: Nat) {
    (x * y).pow(n) = x.pow(n) * y.pow(n)
} by {
    define p(k: Nat) -> Bool {
        (x * y).pow(k) = x.pow(k) * y.pow(k)
    }

    (x * y).pow(Nat.0) = Real.1
    x.pow(Nat.0) = Real.1
    y.pow(Nat.0) = Real.1
    Real.1 * Real.1 = Real.1
    (x * y).pow(Nat.0) = x.pow(Nat.0) * y.pow(Nat.0)
    p(Nat.0)

    forall(k: Nat) {
        real_pow_mul_distrib_step(x, y, k)
        (x * y).pow(k) = x.pow(k) * y.pow(k) implies (x * y).pow(k.suc) = x.pow(k.suc) * y.pow(k.suc)
        if p(k) {
            p(k.suc)
        }
    }

    p(n)
}

/// An even power of -x equals the same power of x.
theorem neg_pow_even(x: Real, n: Nat) {
    (-x).pow(Nat.2 * n) = x.pow(Nat.2 * n)
} by {
    mul_neg_one_left(x)
    -Real.1 * x = -x
    (-x).pow(Nat.2 * n) = (-Real.1 * x).pow(Nat.2 * n)
    real_pow_mul_distrib(-Real.1, x, Nat.2 * n)
    (-Real.1 * x).pow(Nat.2 * n) = (-Real.1).pow(Nat.2 * n) * x.pow(Nat.2 * n)
    alternating_sign_eq_neg_one_pow[Real](Nat.2 * n)
    alternating_sign[Real](Nat.2 * n) = (-Real.1).pow(Nat.2 * n)
    (-Real.1).pow(Nat.2 * n) = alternating_sign[Real](Nat.2 * n)
    alternating_sign_double(n)
    alternating_sign[Real](Nat.2 * n) = Real.1
    (-Real.1).pow(Nat.2 * n) = Real.1
    (-Real.1 * x).pow(Nat.2 * n) = Real.1 * x.pow(Nat.2 * n)
    Real.1 * x.pow(Nat.2 * n) = x.pow(Nat.2 * n)
    (-x).pow(Nat.2 * n) = x.pow(Nat.2 * n)
}

/// An odd power of -x is the negation of the same power of x.
theorem neg_pow_odd(x: Real, n: Nat) {
    (-x).pow(Nat.2 * n + Nat.1) = -x.pow(Nat.2 * n + Nat.1)
} by {
    mul_neg_one_left(x)
    -Real.1 * x = -x
    (-x).pow(Nat.2 * n + Nat.1) = (-Real.1 * x).pow(Nat.2 * n + Nat.1)
    real_pow_mul_distrib(-Real.1, x, Nat.2 * n + Nat.1)
    (-Real.1 * x).pow(Nat.2 * n + Nat.1) = (-Real.1).pow(Nat.2 * n + Nat.1) * x.pow(Nat.2 * n + Nat.1)
    alternating_sign_eq_neg_one_pow[Real](Nat.2 * n + Nat.1)
    alternating_sign[Real](Nat.2 * n + Nat.1) = (-Real.1).pow(Nat.2 * n + Nat.1)
    (-Real.1).pow(Nat.2 * n + Nat.1) = alternating_sign[Real](Nat.2 * n + Nat.1)
    alternating_sign_double_suc(n)
    alternating_sign[Real](Nat.2 * n + Nat.1) = -Real.1
    (-Real.1).pow(Nat.2 * n + Nat.1) = -Real.1
    (-Real.1 * x).pow(Nat.2 * n + Nat.1) = (-Real.1) * x.pow(Nat.2 * n + Nat.1)
    mul_neg_one_left(x.pow(Nat.2 * n + Nat.1))
    (-Real.1) * x.pow(Nat.2 * n + Nat.1) = -x.pow(Nat.2 * n + Nat.1)
    (-x).pow(Nat.2 * n + Nat.1) = -x.pow(Nat.2 * n + Nat.1)
}

/// The cosine terms are even.
theorem cos_term_neg(x: Real, n: Nat) {
    cos_term(-x, n) = cos_term(x, n)
} by {
    neg_pow_even(x, n)
    (-x).pow(Nat.2 * n) = x.pow(Nat.2 * n)
    cos_term(-x, n) = alternating_sign[Real](n) * (-x).pow(Nat.2 * n) / Real.from_rat(Rat.from_nat((Nat.2 * n).factorial))
    alternating_sign[Real](n) * (-x).pow(Nat.2 * n) / Real.from_rat(Rat.from_nat((Nat.2 * n).factorial)) =
        alternating_sign[Real](n) * x.pow(Nat.2 * n) / Real.from_rat(Rat.from_nat((Nat.2 * n).factorial))
    cos_term(-x, n) = cos_term(x, n)
}

/// The sine terms are odd.
theorem sin_term_neg(x: Real, n: Nat) {
    sin_term(-x, n) = -sin_term(x, n)
} by {
    neg_pow_odd(x, n)
    (-x).pow(Nat.2 * n + Nat.1) = -x.pow(Nat.2 * n + Nat.1)
    sin_term(-x, n) = alternating_sign[Real](n) * (-x).pow(Nat.2 * n + Nat.1) / Real.from_rat(Rat.from_nat((Nat.2 * n + Nat.1).factorial))
    alternating_sign[Real](n) * (-x).pow(Nat.2 * n + Nat.1) / Real.from_rat(Rat.from_nat((Nat.2 * n + Nat.1).factorial)) =
        alternating_sign[Real](n) * (-x.pow(Nat.2 * n + Nat.1)) / Real.from_rat(Rat.from_nat((Nat.2 * n + Nat.1).factorial))
    mul_neg_right(alternating_sign[Real](n), x.pow(Nat.2 * n + Nat.1))
    alternating_sign[Real](n) * -x.pow(Nat.2 * n + Nat.1) = -(alternating_sign[Real](n) * x.pow(Nat.2 * n + Nat.1))
    alternating_sign[Real](n) * (-x.pow(Nat.2 * n + Nat.1)) / Real.from_rat(Rat.from_nat((Nat.2 * n + Nat.1).factorial)) =
        -(alternating_sign[Real](n) * x.pow(Nat.2 * n + Nat.1)) / Real.from_rat(Rat.from_nat((Nat.2 * n + Nat.1).factorial))
    neg_div(alternating_sign[Real](n) * x.pow(Nat.2 * n + Nat.1), Real.from_rat(Rat.from_nat((Nat.2 * n + Nat.1).factorial)))
    -(alternating_sign[Real](n) * x.pow(Nat.2 * n + Nat.1)) / Real.from_rat(Rat.from_nat((Nat.2 * n + Nat.1).factorial)) =
        -sin_term(x, n)
    sin_term(-x, n) = -sin_term(x, n)
}

/// Negation commutes with partial sums.
theorem partial_neg_seq(f: Nat -> Real, n: Nat) {
    partial(neg_seq(f), n) = -partial(f, n)
} by {
    partial_scalar_mul(-Real.1, f, n)
    -Real.1 * partial(f, n) = partial(mul_fn(-Real.1, f), n)
    forall(m: Nat) {
        neg_seq(f)(m) = -Real.1 * f(m)
        mul_fn(-Real.1, f)(m) = -Real.1 * f(m)
        neg_seq(f)(m) = mul_fn(-Real.1, f)(m)
    }
    neg_seq(f) = mul_fn(-Real.1, f)
    partial(neg_seq(f), n) = partial(mul_fn(-Real.1, f), n)
    partial(mul_fn(-Real.1, f), n) = -Real.1 * partial(f, n)
    mul_neg_one_left(partial(f, n))
    -Real.1 * partial(f, n) = -partial(f, n)
    partial(neg_seq(f), n) = -partial(f, n)
}

/// Cosine is even.
theorem cos_neg(x: Real) {
    (-x).cos = x.cos
} by {
    forall(n: Nat) {
        cos_term_neg(x, n)
        cos_term(-x, n) = cos_term(x, n)
    }
    cos_term(-x) = cos_term(x)
    partial(cos_term(-x)) = partial(cos_term(x))
    limit(partial(cos_term(-x))) = limit(partial(cos_term(x)))
    (-x).cos = limit(partial(cos_term(-x)))
    (-x).cos = limit(partial(cos_term(x)))
    x.cos = limit(partial(cos_term(x)))
    (-x).cos = x.cos
}

/// Sine is odd.
theorem sin_neg(x: Real) {
    (-x).sin = -x.sin
} by {
    forall(n: Nat) {
        sin_term_neg(x, n)
        sin_term(-x, n) = -sin_term(x, n)
        neg_seq(sin_term(x))(n) = -Real.1 * sin_term(x, n)
        mul_neg_one_left(sin_term(x, n))
        -Real.1 * sin_term(x, n) = -sin_term(x, n)
        neg_seq(sin_term(x))(n) = -sin_term(x, n)
        sin_term(-x, n) = neg_seq(sin_term(x))(n)
    }
    sin_term(-x) = neg_seq(sin_term(x))
    partial(sin_term(-x)) = partial(neg_seq(sin_term(x)))
    forall(n: Nat) {
        partial_neg_seq(sin_term(x), n)
        partial(neg_seq(sin_term(x)), n) = -partial(sin_term(x), n)
        neg_seq(partial(sin_term(x)))(n) = -Real.1 * partial(sin_term(x), n)
        mul_neg_one_left(partial(sin_term(x), n))
        -Real.1 * partial(sin_term(x), n) = -partial(sin_term(x), n)
        neg_seq(partial(sin_term(x)))(n) = -partial(sin_term(x), n)
        partial(neg_seq(sin_term(x)), n) = neg_seq(partial(sin_term(x)))(n)
    }
    partial(neg_seq(sin_term(x))) = neg_seq(partial(sin_term(x)))
    partial(sin_term(-x)) = neg_seq(partial(sin_term(x)))
    sin_term_abs_converges(x)
    absolutely_converges(sin_term(x))
    absolutely_converges_imp_converges(sin_term(x))
    converges(partial(sin_term(x)))
    limit_neg_seq(partial(sin_term(x)))
    limit(neg_seq(partial(sin_term(x)))) = -limit(partial(sin_term(x)))
    limit(partial(sin_term(-x))) = limit(neg_seq(partial(sin_term(x))))
    limit(partial(sin_term(-x))) = -limit(partial(sin_term(x)))
    (-x).sin = limit(partial(sin_term(-x)))
    (-x).sin = -limit(partial(sin_term(x)))
    x.sin = limit(partial(sin_term(x)))
    (-x).sin = -x.sin
}

// Stretch goals, not yet proved: these need a general series-product (Cauchy
// convolution) API for the sine and cosine series, which is not yet available.
// // The Pythagorean identity.
// theorem sin_sq_add_cos_sq(x: Real) {
//     x.sin.pow(Nat.2) + x.cos.pow(Nat.2) = Real.1
// }
// // The sine addition formula.
// theorem sin_add(x: Real, y: Real) {
//     (x + y).sin = x.sin * y.cos + x.cos * y.sin
// }
// // The cosine addition formula.
// theorem cos_add(x: Real, y: Real) {
//     (x + y).cos = x.cos * y.cos - x.sin * y.sin
// }
