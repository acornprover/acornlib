from real.continuity_base import Real, continuous, continuous_at, continuous_condition
from real.continuity_composition import continuous_at_delta, continuous_imp_continuous_at
from real.prod_seq import abs_le_abs_add_eps

/// A function continuous at a point is locally bounded in absolute value there.
theorem continuous_at_local_abs_bound(f: Real -> Real, x: Real) {
    continuous_at(f, x) implies exists(delta: Real, bound: Real) {
        delta.is_positive and bound.is_positive and forall(y: Real) {
            y.is_close(x, delta) implies f(y).abs <= bound
        }
    }
} by {
    Real.1.is_positive
    continuous_at_delta(f, x, Real.1)
    let delta: Real satisfy {
        delta.is_positive and continuous_condition(f, x, delta, Real.1)
    }
    let bound = f(x).abs + Real.1
    not f(x).abs.is_negative
    f(x).abs >= Real.0
    f(x).abs + Real.1 >= Real.0 + Real.1
    Real.0 + Real.1 = Real.1
    f(x).abs + Real.1 >= Real.1
    Real.1 > Real.0
    bound > Real.0
    bound.is_positive
    continuous_condition(f, x, delta, Real.1) = forall(y: Real) {
        y.is_close(x, delta) implies f(y).is_close(f(x), Real.1)
    }
    forall(y: Real) {
        if y.is_close(x, delta) {
            f(y).is_close(f(x), Real.1)
            abs_le_abs_add_eps(f(y), f(x), Real.1)
            f(y).abs <= f(x).abs + Real.1
            f(y).abs <= bound
        }
    }
    exists(delta2: Real, bound2: Real) {
        delta2.is_positive and bound2.is_positive and forall(y: Real) {
            y.is_close(x, delta2) implies f(y).abs <= bound2
        }
    }
}

/// A continuous real function is locally bounded at every real point.
theorem continuous_local_abs_bound(f: Real -> Real, x: Real) {
    continuous(f) implies exists(delta: Real, bound: Real) {
        delta.is_positive and bound.is_positive and forall(y: Real) {
            y.is_close(x, delta) implies f(y).abs <= bound
        }
    }
} by {
    continuous_imp_continuous_at(f, x)
    continuous_at_local_abs_bound(f, x)
}
