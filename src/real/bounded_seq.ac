/// Boundedness of real-valued sequences.
from nat import Nat, lte_antisymm
from real.real_base import Real, lte_abs, abs_gte_zero
from real.real_seq import converges, converges_to, tail_bound, converges_imp_converges_to, tail_bound_implies_is_close
from real.prod_seq import abs_le_abs_add_eps

/// True if the absolute values of a real sequence are uniformly bounded.
define is_bounded_seq(a: Nat -> Real) -> Bool {
    exists(bound: Real) {
        forall(n: Nat) {
            a(n).abs < bound
        }
    }
}

/// Helper: a real strictly larger than two given reals.
theorem real_strict_upper_bound_pair(a: Real, b: Real) {
    exists(c: Real) {
        a < c and b < c
    }
} by {
    let c: Real = a.abs + b.abs + Real.1
    a <= a.abs
    b <= b.abs
    a.abs >= Real.0
    b.abs >= Real.0
    Real.1.is_positive
    a.abs + Real.0 <= a.abs + b.abs
    a.abs <= a.abs + b.abs
    b.abs + Real.0 <= b.abs + a.abs
    b.abs <= a.abs + b.abs
    a.abs + b.abs < a.abs + b.abs + Real.1
    a.abs + b.abs < c
    a.abs < c
    b.abs < c
    a < c
    b < c
}

/// Any finite prefix of a real sequence has a uniform absolute bound.
theorem finite_seq_real_abs_bounded(a: Nat -> Real, n: Nat) {
    exists(bound: Real) {
        forall(i: Nat) {
            i <= n implies a(i).abs < bound
        }
    }
} by {
    define p(k: Nat) -> Bool {
        exists(bound: Real) {
            forall(i: Nat) {
                i <= k implies a(i).abs < bound
            }
        }
    }
    let zero_bound: Real = a(Nat.0).abs + Real.1
    Real.1.is_positive
    a(Nat.0).abs < zero_bound
    forall(i: Nat) {
        if i <= Nat.0 {
            i = Nat.0
            a(i).abs < zero_bound
        }
    }
    p(Nat.0)
    forall(k: Nat) {
        if p(k) {
            let base_bound: Real satisfy {
                forall(i: Nat) {
                    i <= k implies a(i).abs < base_bound
                }
            }
            real_strict_upper_bound_pair(base_bound, a(k.suc).abs)
            let next_bound: Real satisfy {
                base_bound < next_bound and a(k.suc).abs < next_bound
            }
            forall(i: Nat) {
                if i <= k.suc {
                    if i <= k {
                        a(i).abs < base_bound
                        a(i).abs < next_bound
                    } else {
                        not i <= k
                        k < i
                        k.suc <= i
                        i <= k.suc
                        i = k.suc
                        a(i).abs < next_bound
                    }
                }
            }
            p(k.suc)
        }
    }
    p(n)
}

/// A real sequence convergent to a limit is bounded.
theorem converges_to_imp_bounded_seq(a: Nat -> Real, x: Real) {
    converges_to(a, x) implies is_bounded_seq(a)
} by {
    if converges_to(a, x) {
        Real.1.is_positive
        converges_to(a, x) = forall(eps: Real) {
            eps.is_positive implies exists(m: Nat) {
                tail_bound(a, x, m, eps)
            }
        }
        Real.1.is_positive implies exists(m: Nat) {
            tail_bound(a, x, m, Real.1)
        }
        let n: Nat satisfy {
            tail_bound(a, x, n, Real.1)
        }
        finite_seq_real_abs_bounded(a, n)
        let prefix_bound: Real satisfy {
            forall(i: Nat) {
                i <= n implies a(i).abs < prefix_bound
            }
        }
        let tail_cap: Real = x.abs + Real.1
        forall(i: Nat) {
            if n <= i {
                tail_bound_implies_is_close(a, x, n, Real.1, i)
                a(i).is_close(x, Real.1)
                abs_le_abs_add_eps(a(i), x, Real.1)
                a(i).abs <= x.abs + Real.1
                a(i).abs <= tail_cap
            }
        }
        real_strict_upper_bound_pair(prefix_bound, tail_cap)
        let bound: Real satisfy {
            prefix_bound < bound and tail_cap < bound
        }
        forall(i: Nat) {
            if n <= i {
                a(i).abs <= tail_cap
                a(i).abs < bound
            } else {
                i <= n
                a(i).abs < prefix_bound
                a(i).abs < bound
            }
        }
    }
}

/// A convergent real sequence is bounded.
theorem converges_imp_bounded_seq(a: Nat -> Real) {
    converges(a) implies is_bounded_seq(a)
} by {
    if converges(a) {
        converges_imp_converges_to(a)
        let x: Real satisfy {
            converges_to(a, x)
        }
        converges_to_imp_bounded_seq(a, x)
    }
}
