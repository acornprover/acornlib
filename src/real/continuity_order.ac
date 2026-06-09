from real.continuity_base import Real, continuous, continuous_at, continuous_condition
from real.continuity_composition import continuous_at_delta
from real.real_base import close_imp_bounds

/// A local continuity condition applies to any point in its radius.
theorem continuous_condition_apply_local(
    f: Real -> Real, x: Real, delta: Real, eps: Real, y: Real
) {
    continuous_condition(f, x, delta, eps) and y.is_close(x, delta)
    implies f(y).is_close(f(x), eps)
} by {
    if continuous_condition(f, x, delta, eps) and y.is_close(x, delta) {
        continuous_condition(f, x, delta, eps)
        (forall(z: Real) {
            not z.is_close(x, delta) or f(z).is_close(f(x), eps)
        } = true)
        function(z: Real) {
            not (z.is_close(x, delta) and not f(z).is_close(f(x), eps))
        }(y)
        f(y).is_close(f(x), eps)
    }
}

/// A local upper-target neighborhood condition applies to each point in its radius.
theorem lt_target_neighborhood_apply(
    f: Real -> Real, x: Real, target: Real, delta: Real, y: Real
) {
    (forall(z: Real) {
        z.is_close(x, delta) implies f(z) < target
    }) and y.is_close(x, delta) implies f(y) < target
} by {
    if (forall(z: Real) {
        z.is_close(x, delta) implies f(z) < target
    }) and y.is_close(x, delta) {
        (forall(z: Real) {
            not z.is_close(x, delta) or f(z) < target
        } = true)
        function(z: Real) {
            not (z.is_close(x, delta) and not f(z) < target)
        }(y)
        f(y) < target
    }
}

/// A local lower-target neighborhood condition applies to each point in its radius.
theorem target_lt_neighborhood_apply(
    f: Real -> Real, x: Real, target: Real, delta: Real, y: Real
) {
    (forall(z: Real) {
        z.is_close(x, delta) implies target < f(z)
    }) and y.is_close(x, delta) implies target < f(y)
} by {
    if (forall(z: Real) {
        z.is_close(x, delta) implies target < f(z)
    }) and y.is_close(x, delta) {
        (forall(z: Real) {
            not z.is_close(x, delta) or target < f(z)
        } = true)
        function(z: Real) {
            not (z.is_close(x, delta) and not target < f(z))
        }(y)
        target < f(y)
    }
}

/// A continuous real function is continuous at each point.
theorem continuous_at_of_continuous_local(f: Real -> Real, x: Real) {
    continuous(f) implies continuous_at(f, x)
} by {
    if continuous(f) {
        continuous(f) = forall(y: Real) {
            continuous_at(f, y)
        }
        continuous_at(f, x)
    }
}

lemma continuous_at_eventually_close_local(f: Real -> Real, x: Real, eps: Real) {
    continuous_at(f, x) and eps.is_positive implies exists(delta: Real) {
        delta.is_positive and forall(y: Real) {
            y.is_close(x, delta) implies f(y).is_close(f(x), eps)
        }
    }
} by {
    if continuous_at(f, x) and eps.is_positive {
        continuous_at_delta(f, x, eps)
        let delta: Real satisfy {
            delta.is_positive and continuous_condition(f, x, delta, eps)
        }
        forall(y: Real) {
            if y.is_close(x, delta) {
                continuous_condition_apply_local(f, x, delta, eps, y)
                f(y).is_close(f(x), eps)
            }
        }
        exists(delta2: Real) {
            delta2.is_positive and forall(y2: Real) {
                y2.is_close(x, delta2) implies f(y2).is_close(f(x), eps)
            }
        }
    }
}

/// Values below a continuity point remain below a strict upper target nearby.
theorem continuous_at_lt_target_neighborhood(f: Real -> Real, x: Real, target: Real) {
    continuous_at(f, x) and f(x) < target implies exists(delta: Real) {
        delta.is_positive and forall(y: Real) {
            y.is_close(x, delta) implies f(y) < target
        }
    }
} by {
    if continuous_at(f, x) and f(x) < target {
        let eps = target - f(x)
        eps.is_positive
        f(x) + eps = f(x) + (target - f(x))
        target - f(x) + f(x) = target
        f(x) + (target - f(x)) = target - f(x) + f(x)
        f(x) + (target - f(x)) = target
        f(x) + eps = target
        continuous_at_delta(f, x, eps)
        let delta: Real satisfy {
            delta.is_positive and continuous_condition(f, x, delta, eps)
        }
        forall(y: Real) {
            if y.is_close(x, delta) {
                continuous_condition_apply_local(f, x, delta, eps, y)
                f(y).is_close(f(x), eps)
                close_imp_bounds(f(y), f(x), eps)
                f(y) < f(x) + eps
                f(y) < target
            }
        }
        delta.is_positive and forall(y2: Real) {
            y2.is_close(x, delta) implies f(y2) < target
        }
        exists(delta2: Real) {
            delta2.is_positive and forall(y2: Real) {
                y2.is_close(x, delta2) implies f(y2) < target
            }
        }
    }
}

/// Values of a continuous function below a strict upper target remain below nearby.
theorem continuous_lt_target_neighborhood(f: Real -> Real, x: Real, target: Real) {
    continuous(f) and f(x) < target implies exists(delta: Real) {
        delta.is_positive and forall(y: Real) {
            y.is_close(x, delta) implies f(y) < target
        }
    }
} by {
    if continuous(f) and f(x) < target {
        continuous_at_of_continuous_local(f, x)
        continuous_at_lt_target_neighborhood(f, x, target)
        let delta: Real satisfy {
            delta.is_positive and forall(y: Real) {
                y.is_close(x, delta) implies f(y) < target
            }
        }
        exists(delta2: Real) {
            delta2.is_positive and forall(y2: Real) {
                y2.is_close(x, delta2) implies f(y2) < target
            }
        }
    }
}

/// Values above a continuity point remain above a strict lower target nearby.
theorem continuous_at_target_lt_neighborhood(f: Real -> Real, x: Real, target: Real) {
    continuous_at(f, x) and target < f(x) implies exists(delta: Real) {
        delta.is_positive and forall(y: Real) {
            y.is_close(x, delta) implies target < f(y)
        }
    }
} by {
    if continuous_at(f, x) and target < f(x) {
        let eps = f(x) - target
        eps.is_positive
        f(x) - eps = f(x) - (f(x) - target)
        f(x) - (f(x) - target) = f(x) + -(f(x) - target)
        -(f(x) - target) = target - f(x)
        f(x) + -(f(x) - target) = f(x) + (target - f(x))
        target - f(x) + f(x) = target
        f(x) + (target - f(x)) = target - f(x) + f(x)
        f(x) + (target - f(x)) = target
        f(x) - (f(x) - target) = target
        f(x) - eps = target
        continuous_at_delta(f, x, eps)
        let delta: Real satisfy {
            delta.is_positive and continuous_condition(f, x, delta, eps)
        }
        forall(y: Real) {
            if y.is_close(x, delta) {
                continuous_condition_apply_local(f, x, delta, eps, y)
                f(y).is_close(f(x), eps)
                close_imp_bounds(f(y), f(x), eps)
                f(x) - eps < f(y)
                target < f(y)
            }
        }
        delta.is_positive and forall(y2: Real) {
            y2.is_close(x, delta) implies target < f(y2)
        }
        exists(delta2: Real) {
            delta2.is_positive and forall(y2: Real) {
                y2.is_close(x, delta2) implies target < f(y2)
            }
        }
    }
}

/// Values of a continuous function above a strict lower target remain above nearby.
theorem continuous_target_lt_neighborhood(f: Real -> Real, x: Real, target: Real) {
    continuous(f) and target < f(x) implies exists(delta: Real) {
        delta.is_positive and forall(y: Real) {
            y.is_close(x, delta) implies target < f(y)
        }
    }
} by {
    if continuous(f) and target < f(x) {
        continuous_at_of_continuous_local(f, x)
        continuous_at_target_lt_neighborhood(f, x, target)
        let delta: Real satisfy {
            delta.is_positive and forall(y: Real) {
                y.is_close(x, delta) implies target < f(y)
            }
        }
        exists(delta2: Real) {
            delta2.is_positive and forall(y2: Real) {
                y2.is_close(x, delta2) implies target < f(y2)
            }
        }
    }
}

/// Near a continuity point where the value is negative, the function remains negative.
theorem continuous_at_negative_neighborhood(f: Real -> Real, x: Real) {
    continuous_at(f, x) and f(x) < Real.0 implies exists(delta: Real) {
        delta.is_positive and forall(y: Real) {
            y.is_close(x, delta) implies f(y) < Real.0
        }
    }
} by {
    if continuous_at(f, x) and f(x) < Real.0 {
        continuous_at_lt_target_neighborhood(f, x, Real.0)
        let delta: Real satisfy {
            delta.is_positive and forall(y: Real) {
                y.is_close(x, delta) implies f(y) < Real.0
            }
        }
        exists(delta2: Real) {
            delta2.is_positive and forall(y2: Real) {
                y2.is_close(x, delta2) implies f(y2) < Real.0
            }
        }
    }
}

/// Near a continuity point where the value is positive, the function remains positive.
theorem continuous_at_positive_neighborhood(f: Real -> Real, x: Real) {
    continuous_at(f, x) and Real.0 < f(x) implies exists(delta: Real) {
        delta.is_positive and forall(y: Real) {
            y.is_close(x, delta) implies Real.0 < f(y)
        }
    }
} by {
    if continuous_at(f, x) and Real.0 < f(x) {
        continuous_at_target_lt_neighborhood(f, x, Real.0)
        let delta: Real satisfy {
            delta.is_positive and forall(y: Real) {
                y.is_close(x, delta) implies Real.0 < f(y)
            }
        }
        exists(delta2: Real) {
            delta2.is_positive and forall(y2: Real) {
                y2.is_close(x, delta2) implies Real.0 < f(y2)
            }
        }
    }
}

/// A continuous function that is negative at a point is negative nearby.
theorem continuous_negative_neighborhood(f: Real -> Real, x: Real) {
    continuous(f) and f(x) < Real.0 implies exists(delta: Real) {
        delta.is_positive and forall(y: Real) {
            y.is_close(x, delta) implies f(y) < Real.0
        }
    }
} by {
    if continuous(f) and f(x) < Real.0 {
        continuous_lt_target_neighborhood(f, x, Real.0)
        let delta: Real satisfy {
            delta.is_positive and forall(y: Real) {
                y.is_close(x, delta) implies f(y) < Real.0
            }
        }
        exists(delta2: Real) {
            delta2.is_positive and forall(y2: Real) {
                y2.is_close(x, delta2) implies f(y2) < Real.0
            }
        }
    }
}

/// A continuous function that is positive at a point is positive nearby.
theorem continuous_positive_neighborhood(f: Real -> Real, x: Real) {
    continuous(f) and Real.0 < f(x) implies exists(delta: Real) {
        delta.is_positive and forall(y: Real) {
            y.is_close(x, delta) implies Real.0 < f(y)
        }
    }
} by {
    if continuous(f) and Real.0 < f(x) {
        continuous_target_lt_neighborhood(f, x, Real.0)
        let delta: Real satisfy {
            delta.is_positive and forall(y: Real) {
                y.is_close(x, delta) implies Real.0 < f(y)
            }
        }
        exists(delta2: Real) {
            delta2.is_positive and forall(y2: Real) {
                y2.is_close(x, delta2) implies Real.0 < f(y2)
            }
        }
    }
}
