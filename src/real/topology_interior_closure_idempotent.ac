from data.basic.set import Set, compl_of_compl_is_self, double_inclusion, empty_set_compl_is_universal,
    empty_set_is_always_subset, universal_set_compl_is_empty
from real.real_field import Real
from real.topology import closure, closure_subset_of_closed_set, empty_set_is_closed,
    empty_set_is_open, interior, interior_subset, is_closed_set, is_open_set,
    set_subset_closure, universal_set_is_closed, universal_set_is_open
from real.topology_closed_open import complement_of_open_is_closed
from real.topology_closure import closure_is_closed
from real.topology_closure_interior_duality import interior_eq_closure_complement_complement
from real.topology_complements import complement_of_closed_is_open
from real.topology_interior_algebra import open_set_eq_interior

/// The interior of any real set is open.
theorem interior_is_open(s: Set[Real]) {
    is_open_set(interior(s))
} by {
    interior_eq_closure_complement_complement(s)
    interior(s) = closure(s.c).c
    closure_is_closed(s.c)
    is_closed_set(closure(s.c))
    complement_of_closed_is_open(closure(s.c))
    is_open_set(closure(s.c).c)
    is_open_set(interior(s))
}

/// Interior is idempotent.
theorem interior_idempotent(s: Set[Real]) {
    interior(interior(s)) = interior(s)
} by {
    interior_is_open(s)
    is_open_set(interior(s))
    open_set_eq_interior(interior(s))
    interior(s) = interior(interior(s))
    interior(interior(s)) = interior(s)
}

/// A set is open exactly when it equals its interior.
theorem open_set_iff_eq_interior(s: Set[Real]) {
    is_open_set(s) = (s = interior(s))
} by {
    if is_open_set(s) {
        open_set_eq_interior(s)
        s = interior(s)
    }
    if s = interior(s) {
        interior_is_open(s)
        is_open_set(interior(s))
        is_open_set(s)
    }
    is_open_set(s) = (s = interior(s))
}

/// A set is closed exactly when it equals its closure.
theorem closed_set_iff_eq_closure(s: Set[Real]) {
    is_closed_set(s) = (s = closure(s))
} by {
    if is_closed_set(s) {
        set_subset_closure(s)
        closure_subset_of_closed_set(s)
        s.subset(closure(s)) and closure(s).subset(s)
        double_inclusion(s, closure(s))
        s = closure(s)
    }
    if s = closure(s) {
        closure_is_closed(s)
        is_closed_set(closure(s))
        is_closed_set(s)
    }
    is_closed_set(s) = (s = closure(s))
}

/// The complement of an interior is closed.
theorem complement_of_interior_is_closed(s: Set[Real]) {
    is_closed_set(interior(s).c)
} by {
    interior_is_open(s)
    is_open_set(interior(s))
    complement_of_open_is_closed(interior(s))
    is_closed_set(interior(s).c)
}

/// The complement of a closure is open.
theorem complement_of_closure_is_open(s: Set[Real]) {
    is_open_set(closure(s).c)
} by {
    closure_is_closed(s)
    is_closed_set(closure(s))
    complement_of_closed_is_open(closure(s))
    is_open_set(closure(s).c)
}

/// The closure of the empty real set is empty.
theorem closure_empty_real_set {
    closure(Set[Real].empty_set) = Set[Real].empty_set
} by {
    empty_set_is_closed
    is_closed_set(Set[Real].empty_set)
    closure_subset_of_closed_set(Set[Real].empty_set)
    closure(Set[Real].empty_set).subset(Set[Real].empty_set)
    empty_set_is_always_subset[Real](closure(Set[Real].empty_set))
    Set[Real].empty_set.subset(closure(Set[Real].empty_set))
    double_inclusion(closure(Set[Real].empty_set), Set[Real].empty_set)
}

/// The closure of the universal real set is universal.
theorem closure_universal_real_set {
    closure(Set[Real].universal_set) = Set[Real].universal_set
} by {
    universal_set_is_closed
    is_closed_set(Set[Real].universal_set)
    closure_subset_of_closed_set(Set[Real].universal_set)
    closure(Set[Real].universal_set).subset(Set[Real].universal_set)
    set_subset_closure(Set[Real].universal_set)
    Set[Real].universal_set.subset(closure(Set[Real].universal_set))
    double_inclusion(closure(Set[Real].universal_set), Set[Real].universal_set)
}

/// The interior of the empty real set is empty.
theorem interior_empty_real_set {
    interior(Set[Real].empty_set) = Set[Real].empty_set
} by {
    empty_set_is_open
    is_open_set(Set[Real].empty_set)
    open_set_eq_interior(Set[Real].empty_set)
    Set[Real].empty_set = interior(Set[Real].empty_set)
    interior(Set[Real].empty_set) = Set[Real].empty_set
}

/// The interior of the universal real set is universal.
theorem interior_universal_real_set {
    interior(Set[Real].universal_set) = Set[Real].universal_set
} by {
    universal_set_is_open
    is_open_set(Set[Real].universal_set)
    open_set_eq_interior(Set[Real].universal_set)
    Set[Real].universal_set = interior(Set[Real].universal_set)
    interior(Set[Real].universal_set) = Set[Real].universal_set
}

/// A set is open exactly when its complement is closed.
theorem open_set_iff_complement_closed(s: Set[Real]) {
    is_open_set(s) = is_closed_set(s.c)
} by {
    if is_open_set(s) {
        complement_of_open_is_closed(s)
        is_closed_set(s.c)
    }
    if is_closed_set(s.c) {
        complement_of_closed_is_open(s.c)
        is_open_set(s.c.c)
        compl_of_compl_is_self[Real](s)
        s.c.c = s
        is_open_set(s)
    }
    is_open_set(s) = is_closed_set(s.c)
}

/// A set is closed exactly when its complement is open.
theorem closed_set_iff_complement_open(s: Set[Real]) {
    is_closed_set(s) = is_open_set(s.c)
} by {
    if is_closed_set(s) {
        complement_of_closed_is_open(s)
        is_open_set(s.c)
    }
    if is_open_set(s.c) {
        complement_of_open_is_closed(s.c)
        is_closed_set(s.c.c)
        compl_of_compl_is_self[Real](s)
        s.c.c = s
        is_closed_set(s)
    }
    is_closed_set(s) = is_open_set(s.c)
}

/// The complement of the empty real set has universal interior.
theorem interior_complement_empty_real_set {
    interior(Set[Real].empty_set.c) = Set[Real].universal_set
} by {
    empty_set_compl_is_universal[Real]
    Set[Real].empty_set.c = Set[Real].universal_set
    interior_universal_real_set
    interior(Set[Real].universal_set) = Set[Real].universal_set
    interior(Set[Real].empty_set.c) = Set[Real].universal_set
}

/// The complement of the universal real set has empty interior.
theorem interior_complement_universal_real_set {
    interior(Set[Real].universal_set.c) = Set[Real].empty_set
} by {
    universal_set_compl_is_empty[Real]
    Set[Real].universal_set.c = Set[Real].empty_set
    interior_empty_real_set
    interior(Set[Real].empty_set) = Set[Real].empty_set
    interior(Set[Real].universal_set.c) = Set[Real].empty_set
}

/// The closure of the complement of the empty real set is universal.
theorem closure_complement_empty_real_set {
    closure(Set[Real].empty_set.c) = Set[Real].universal_set
} by {
    empty_set_compl_is_universal[Real]
    Set[Real].empty_set.c = Set[Real].universal_set
    closure_universal_real_set
    closure(Set[Real].universal_set) = Set[Real].universal_set
    closure(Set[Real].empty_set.c) = Set[Real].universal_set
}

/// The closure of the complement of the universal real set is empty.
theorem closure_complement_universal_real_set {
    closure(Set[Real].universal_set.c) = Set[Real].empty_set
} by {
    universal_set_compl_is_empty[Real]
    Set[Real].universal_set.c = Set[Real].empty_set
    closure_empty_real_set
    closure(Set[Real].empty_set) = Set[Real].empty_set
    closure(Set[Real].universal_set.c) = Set[Real].empty_set
}

/// The closure of an interior is contained in the closure.
theorem closure_interior_subset_closure(s: Set[Real]) {
    closure(interior(s)).subset(closure(s))
} by {
    interior_subset(s)
    interior(s).subset(s)
    from real.topology import closure_mono
    closure_mono(interior(s), s)
    closure(interior(s)).subset(closure(s))
}

/// The interior is contained in the interior of the closure.
theorem interior_subset_interior_closure(s: Set[Real]) {
    interior(s).subset(interior(closure(s)))
} by {
    set_subset_closure(s)
    s.subset(closure(s))
    from real.topology_interior_algebra import interior_mono
    interior_mono(s, closure(s))
    interior(s).subset(interior(closure(s)))
}

/// The interior of a closure is open.
theorem interior_closure_is_open(s: Set[Real]) {
    is_open_set(interior(closure(s)))
} by {
    interior_is_open(closure(s))
}

/// The closure of an interior is closed.
theorem closure_interior_is_closed(s: Set[Real]) {
    is_closed_set(closure(interior(s)))
} by {
    closure_is_closed(interior(s))
}
