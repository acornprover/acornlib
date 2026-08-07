/// Cauchy preservation and canonical-limit consumers for sequence reindexing.
///
/// This module intentionally does not define growth-rate notation or another
/// generic eventual predicate layer.  It packages
/// shallow Cauchy consequences of the accepted convergence and Nat-eventual
/// sequence transport APIs.

from data.basic.functions import compose
from nat import Nat
from data.basic.set import Set
from real.real_field import Real
from real.real_seq import converges, converges_to, limit
from real.limits import tends_to_infinity, is_subsequence_index, subsequence,
    converges_compose_tends_to_infinity, converges_compose_add,
    converges_subsequence, converges_subsequence_add
from real.cauchy_criterion import is_cauchy_seq, cauchy_imp_converges,
    converges_to_imp_cauchy
from real.sequence_set_membership import seq_eventually_in_real_set
from real.sequence_eventual_tail_transport import seq_eventually_in_real_set_compose_tends_to_infinity,
    seq_eventually_in_real_set_shift_add,
    seq_eventually_in_real_set_subsequence
from real.cauchy_topology import cauchy_seq_eventually_in_imp_limit_in_closure,
    closed_set_contains_eventual_cauchy_seq_limit
from real.topology import closure, is_closed_set

/// Reindexing a Cauchy sequence along any map tending to infinity preserves Cauchy-ness.
theorem cauchy_seq_compose_tends_to_infinity(a: Nat -> Real, f: Nat -> Nat) {
    is_cauchy_seq(a) and tends_to_infinity(f) implies is_cauchy_seq(compose(a, f))
} by {
    if is_cauchy_seq(a) and tends_to_infinity(f) {
        cauchy_imp_converges(a)
        converges(a)
        converges_compose_tends_to_infinity(a, f)
        converges_to(compose(a, f), limit(a))
        converges_to_imp_cauchy(compose(a, f), limit(a))
        is_cauchy_seq(compose(a, f))
    }
}

/// Discarding a finite prefix of a Cauchy sequence preserves Cauchy-ness.
theorem cauchy_seq_shift_add(a: Nat -> Real, k: Nat) {
    is_cauchy_seq(a) implies is_cauchy_seq(compose(a, k.add))
} by {
    if is_cauchy_seq(a) {
        cauchy_imp_converges(a)
        converges(a)
        converges_compose_add(a, k)
        converges_to(compose(a, k.add), limit(a))
        converges_to_imp_cauchy(compose(a, k.add), limit(a))
        is_cauchy_seq(compose(a, k.add))
    }
}

/// Taking a selected library subsequence of a Cauchy sequence preserves Cauchy-ness.
theorem cauchy_seq_subsequence(a: Nat -> Real, f: Nat -> Nat) {
    is_cauchy_seq(a) and is_subsequence_index(f) implies is_cauchy_seq(subsequence(a, f))
} by {
    if is_cauchy_seq(a) and is_subsequence_index(f) {
        cauchy_imp_converges(a)
        converges(a)
        converges_subsequence(a, f)
        converges_to(subsequence(a, f), limit(a))
        converges_to_imp_cauchy(subsequence(a, f), limit(a))
        is_cauchy_seq(subsequence(a, f))
    }
}

/// Taking the tail subsequence `n ↦ k + n` of a Cauchy sequence preserves Cauchy-ness.
theorem cauchy_seq_tail_subsequence(a: Nat -> Real, k: Nat) {
    is_cauchy_seq(a) implies is_cauchy_seq(subsequence(a, k.add))
} by {
    if is_cauchy_seq(a) {
        cauchy_imp_converges(a)
        converges(a)
        converges_subsequence_add(a, k)
        converges_to(subsequence(a, k.add), limit(a))
        converges_to_imp_cauchy(subsequence(a, k.add), limit(a))
        is_cauchy_seq(subsequence(a, k.add))
    }
}

/// If a Cauchy sequence is eventually in `s`, then any infinity-tending reindexing
/// has its canonical limit in `closure(s)`.
theorem eventual_compose_cauchy_limit_in_closure(
    s: Set[Real],
    a: Nat -> Real,
    f: Nat -> Nat
) {
    seq_eventually_in_real_set(s, a) and is_cauchy_seq(a) and tends_to_infinity(f)
    implies closure(s).contains(limit(compose(a, f)))
} by {
    if seq_eventually_in_real_set(s, a) and is_cauchy_seq(a) and tends_to_infinity(f) {
        seq_eventually_in_real_set_compose_tends_to_infinity(s, a, f)
        seq_eventually_in_real_set(s, compose(a, f))
        cauchy_seq_compose_tends_to_infinity(a, f)
        is_cauchy_seq(compose(a, f))
        cauchy_seq_eventually_in_imp_limit_in_closure(s, compose(a, f))
        closure(s).contains(limit(compose(a, f)))
    }
}

/// Closed-set variant for an infinity-tending reindexing of an eventual Cauchy sequence.
theorem closed_set_contains_eventual_compose_cauchy_limit(
    s: Set[Real],
    a: Nat -> Real,
    f: Nat -> Nat
) {
    is_closed_set(s) and seq_eventually_in_real_set(s, a) and is_cauchy_seq(a) and tends_to_infinity(f)
    implies s.contains(limit(compose(a, f)))
} by {
    if is_closed_set(s) and seq_eventually_in_real_set(s, a) and is_cauchy_seq(a) and tends_to_infinity(f) {
        seq_eventually_in_real_set_compose_tends_to_infinity(s, a, f)
        seq_eventually_in_real_set(s, compose(a, f))
        cauchy_seq_compose_tends_to_infinity(a, f)
        is_cauchy_seq(compose(a, f))
        closed_set_contains_eventual_cauchy_seq_limit(s, compose(a, f))
        s.contains(limit(compose(a, f)))
    }
}

/// If a Cauchy sequence is eventually in `s`, then every finite-prefix shift has
/// its canonical limit in `closure(s)`.
theorem eventual_shift_add_cauchy_limit_in_closure(s: Set[Real], a: Nat -> Real, k: Nat) {
    seq_eventually_in_real_set(s, a) and is_cauchy_seq(a)
    implies closure(s).contains(limit(compose(a, k.add)))
} by {
    if seq_eventually_in_real_set(s, a) and is_cauchy_seq(a) {
        seq_eventually_in_real_set_shift_add(s, a, k)
        seq_eventually_in_real_set(s, compose(a, k.add))
        cauchy_seq_shift_add(a, k)
        is_cauchy_seq(compose(a, k.add))
        cauchy_seq_eventually_in_imp_limit_in_closure(s, compose(a, k.add))
        closure(s).contains(limit(compose(a, k.add)))
    }
}

/// Closed-set variant for a finite-prefix shift of an eventual Cauchy sequence.
theorem closed_set_contains_eventual_shift_add_cauchy_limit(s: Set[Real], a: Nat -> Real, k: Nat) {
    is_closed_set(s) and seq_eventually_in_real_set(s, a) and is_cauchy_seq(a)
    implies s.contains(limit(compose(a, k.add)))
} by {
    if is_closed_set(s) and seq_eventually_in_real_set(s, a) and is_cauchy_seq(a) {
        seq_eventually_in_real_set_shift_add(s, a, k)
        seq_eventually_in_real_set(s, compose(a, k.add))
        cauchy_seq_shift_add(a, k)
        is_cauchy_seq(compose(a, k.add))
        closed_set_contains_eventual_cauchy_seq_limit(s, compose(a, k.add))
        s.contains(limit(compose(a, k.add)))
    }
}

/// If a Cauchy sequence is eventually in `s`, then every selected subsequence has
/// its canonical limit in `closure(s)`.
theorem eventual_subsequence_cauchy_limit_in_closure(
    s: Set[Real],
    a: Nat -> Real,
    f: Nat -> Nat
) {
    seq_eventually_in_real_set(s, a) and is_cauchy_seq(a) and is_subsequence_index(f)
    implies closure(s).contains(limit(subsequence(a, f)))
} by {
    if seq_eventually_in_real_set(s, a) and is_cauchy_seq(a) and is_subsequence_index(f) {
        seq_eventually_in_real_set_subsequence(s, a, f)
        seq_eventually_in_real_set(s, subsequence(a, f))
        cauchy_seq_subsequence(a, f)
        is_cauchy_seq(subsequence(a, f))
        cauchy_seq_eventually_in_imp_limit_in_closure(s, subsequence(a, f))
        closure(s).contains(limit(subsequence(a, f)))
    }
}

/// Closed-set variant for a selected subsequence of an eventual Cauchy sequence.
theorem closed_set_contains_eventual_subsequence_cauchy_limit(
    s: Set[Real],
    a: Nat -> Real,
    f: Nat -> Nat
) {
    is_closed_set(s) and seq_eventually_in_real_set(s, a) and is_cauchy_seq(a) and is_subsequence_index(f)
    implies s.contains(limit(subsequence(a, f)))
} by {
    if is_closed_set(s) and seq_eventually_in_real_set(s, a) and is_cauchy_seq(a) and is_subsequence_index(f) {
        seq_eventually_in_real_set_subsequence(s, a, f)
        seq_eventually_in_real_set(s, subsequence(a, f))
        cauchy_seq_subsequence(a, f)
        is_cauchy_seq(subsequence(a, f))
        closed_set_contains_eventual_cauchy_seq_limit(s, subsequence(a, f))
        s.contains(limit(subsequence(a, f)))
    }
}
