from data.basic.set import Set, compl_contains_eq, compl_of_compl_is_self, difference_contains_eq,
    difference_contains_intro, double_inclusion, intersection_comm, intersection_contains_eq,
    intersection_contains_intro, set_ext, union_contains_eq, union_contains_left,
    union_contains_right
from real.real_field import Real
from real.topology import closure, interior, interior_subset, is_closed_set, is_open_set,
    point_in_closure_if_in_set, set_subset_closure
from real.topology_closed_open import intersection_of_closed_is_closed
from real.topology_closure import closure_is_closed
from real.topology_closure_interior_duality import closure_complement_eq_interior_complement,
    interior_complement_eq_closure_complement
from real.topology_interior_algebra import open_set_eq_interior

/// The exterior of a real set.
define exterior(s: Set[Real]) -> Set[Real] {
    interior(s.c)
}

/// The boundary of a real set.
define boundary(s: Set[Real]) -> Set[Real] {
    closure(s).intersection(closure(s.c))
}

/// Membership in the exterior is membership in the interior of the complement.
theorem exterior_contains_eq(s: Set[Real], x: Real) {
    exterior(s).contains(x) = interior(s.c).contains(x)
}

/// Membership in the boundary is membership in both relevant closures.
theorem boundary_contains_eq(s: Set[Real], x: Real) {
    boundary(s).contains(x) = (closure(s).contains(x) and closure(s.c).contains(x))
}

/// A boundary point lies in the closure of the set.
theorem boundary_contains_closure(s: Set[Real], x: Real) {
    boundary(s).contains(x) implies closure(s).contains(x)
} by {
    if boundary(s).contains(x) {
        boundary_contains_eq(s, x)
        closure(s).contains(x)
    }
}

/// A boundary point lies in the closure of the complement.
theorem boundary_contains_closure_complement(s: Set[Real], x: Real) {
    boundary(s).contains(x) implies closure(s.c).contains(x)
} by {
    if boundary(s).contains(x) {
        boundary_contains_eq(s, x)
        closure(s.c).contains(x)
    }
}

/// The boundary is contained in the closure of the set.
theorem boundary_subset_closure(s: Set[Real]) {
    boundary(s).subset(closure(s))
} by {
    forall(x: Real) {
        if boundary(s).contains(x) {
            boundary_contains_closure(s, x)
        }
    }
}

/// The boundary is contained in the closure of the complement.
theorem boundary_subset_closure_complement(s: Set[Real]) {
    boundary(s).subset(closure(s.c))
} by {
    forall(x: Real) {
        if boundary(s).contains(x) {
            boundary_contains_closure_complement(s, x)
        }
    }
}

/// The boundary of a real set is closed.
theorem boundary_is_closed(s: Set[Real]) {
    is_closed_set(boundary(s))
} by {
    closure_is_closed(s)
    closure_is_closed(s.c)
    is_closed_set(closure(s))
    is_closed_set(closure(s.c))
    intersection_of_closed_is_closed(closure(s), closure(s.c))
    is_closed_set(closure(s).intersection(closure(s.c)))
    is_closed_set(boundary(s))
}

/// The exterior is the complement of the closure.
theorem exterior_eq_closure_complement(s: Set[Real]) {
    exterior(s) = closure(s).c
} by {
    interior_complement_eq_closure_complement(s)
    interior(s.c) = closure(s).c
    exterior(s) = closure(s).c
}

/// A point of the exterior is outside the closure.
theorem exterior_not_closure(s: Set[Real], x: Real) {
    exterior(s).contains(x) implies not closure(s).contains(x)
} by {
    if exterior(s).contains(x) {
        exterior_eq_closure_complement(s)
        closure(s).c.contains(x)
        compl_contains_eq(closure(s), x)
        not closure(s).contains(x)
    }
}

/// Interior points are not boundary points.
theorem interior_not_boundary(s: Set[Real], x: Real) {
    interior(s).contains(x) implies not boundary(s).contains(x)
} by {
    if interior(s).contains(x) {
        closure_complement_eq_interior_complement(s)
        closure(s.c) = interior(s).c
        if boundary(s).contains(x) {
            boundary_contains_closure_complement(s, x)
            closure(s.c).contains(x)
            interior(s).c.contains(x)
            compl_contains_eq(interior(s), x)
            not interior(s).contains(x)
            false
        }
    }
}

/// Exterior points are not boundary points.
theorem exterior_not_boundary(s: Set[Real], x: Real) {
    exterior(s).contains(x) implies not boundary(s).contains(x)
} by {
    if exterior(s).contains(x) {
        exterior_not_closure(s, x)
        if boundary(s).contains(x) {
            boundary_contains_closure(s, x)
            closure(s).contains(x)
            false
        }
    }
}

/// The boundary is contained in the complement of the interior.
theorem boundary_subset_interior_complement(s: Set[Real]) {
    boundary(s).subset(interior(s).c)
} by {
    forall(x: Real) {
        if boundary(s).contains(x) {
            if interior(s).contains(x) {
                interior_not_boundary(s, x)
                false
            }
            not interior(s).contains(x)
            compl_contains_eq(interior(s), x)
            interior(s).c.contains(x)
        }
    }
}

/// The interior is contained in the complement of the boundary.
theorem interior_subset_boundary_complement(s: Set[Real]) {
    interior(s).subset(boundary(s).c)
} by {
    forall(x: Real) {
        if interior(s).contains(x) {
            interior_not_boundary(s, x)
            not boundary(s).contains(x)
            compl_contains_eq(boundary(s), x)
            boundary(s).c.contains(x)
        }
    }
}

/// The exterior is contained in the complement of the boundary.
theorem exterior_subset_boundary_complement(s: Set[Real]) {
    exterior(s).subset(boundary(s).c)
} by {
    forall(x: Real) {
        if exterior(s).contains(x) {
            exterior_not_boundary(s, x)
            not boundary(s).contains(x)
            compl_contains_eq(boundary(s), x)
            boundary(s).c.contains(x)
        }
    }
}

/// Set difference is intersection with the complement for real sets.
theorem real_set_difference_eq_intersection_complement(s: Set[Real], t: Set[Real]) {
    s.difference(t) = s.intersection(t.c)
} by {
    let d = s.difference(t)
    let i = s.intersection(t.c)
    forall(x: Real) {
        if d.contains(x) {
            difference_contains_eq(s, t, x)
            s.contains(x)
            not t.contains(x)
            compl_contains_eq(t, x)
            t.c.contains(x)
            intersection_contains_intro(s, t.c, x)
            i.contains(x)
        }
        if i.contains(x) {
            intersection_contains_eq(s, t.c, x)
            s.contains(x)
            t.c.contains(x)
            compl_contains_eq(t, x)
            not t.contains(x)
            difference_contains_intro[Real](s, t, x)
            d.contains(x)
        }
        d.contains(x) = i.contains(x)
    }
    set_ext(d, i)
}

/// The boundary is closure minus interior.
theorem boundary_eq_closure_difference_interior(s: Set[Real]) {
    boundary(s) = closure(s).difference(interior(s))
} by {
    real_set_difference_eq_intersection_complement(closure(s), interior(s))
    closure(s).difference(interior(s)) = closure(s).intersection(interior(s).c)
    closure_complement_eq_interior_complement(s)
    closure(s.c) = interior(s).c
    boundary(s) = closure(s).intersection(closure(s.c))
    boundary(s) = closure(s).intersection(interior(s).c)
    boundary(s) = closure(s).difference(interior(s))
}

/// The closure is contained in the union of the interior and the boundary.
theorem closure_subset_interior_union_boundary(s: Set[Real]) {
    closure(s).subset(interior(s).union(boundary(s)))
} by {
    forall(x: Real) {
        if closure(s).contains(x) {
            if interior(s).contains(x) {
                union_contains_left(interior(s), boundary(s), x)
                interior(s).union(boundary(s)).contains(x)
            } else {
                not interior(s).contains(x)
                compl_contains_eq(interior(s), x)
                interior(s).c.contains(x)
                closure_complement_eq_interior_complement(s)
                closure(s.c) = interior(s).c
                closure(s.c).contains(x)
                intersection_contains_intro(closure(s), closure(s.c), x)
                boundary(s).contains(x)
                union_contains_right(interior(s), boundary(s), x)
                interior(s).union(boundary(s)).contains(x)
            }
        }
    }
}

/// The union of the interior and the boundary is contained in the closure.
theorem interior_union_boundary_subset_closure(s: Set[Real]) {
    interior(s).union(boundary(s)).subset(closure(s))
} by {
    interior_subset(s)
    interior(s).subset(s)
    set_subset_closure(s)
    s.subset(closure(s))
    forall(x: Real) {
        if interior(s).union(boundary(s)).contains(x) {
            union_contains_eq(interior(s), boundary(s), x)
            if interior(s).contains(x) {
                s.contains(x)
                point_in_closure_if_in_set(s, x)
                closure(s).contains(x)
            } else {
                boundary(s).contains(x)
                boundary_contains_closure(s, x)
                closure(s).contains(x)
            }
        }
    }
}

/// The closure is the union of the interior and the boundary.
theorem closure_eq_interior_union_boundary(s: Set[Real]) {
    closure(s) = interior(s).union(boundary(s))
} by {
    closure_subset_interior_union_boundary(s)
    interior_union_boundary_subset_closure(s)
    double_inclusion(closure(s), interior(s).union(boundary(s)))
}

/// The boundary of a complement is the boundary of the original set.
theorem boundary_complement_eq(s: Set[Real]) {
    boundary(s.c) = boundary(s)
} by {
    let left = boundary(s.c)
    let right = boundary(s)
    compl_of_compl_is_self[Real](s)
    s.c.c = s
    forall(x: Real) {
        if left.contains(x) {
            boundary_contains_eq(s.c, x)
            closure(s.c).contains(x)
            closure(s.c.c).contains(x)
            closure(s).contains(x)
            intersection_contains_intro(closure(s), closure(s.c), x)
            right.contains(x)
        }
        if right.contains(x) {
            boundary_contains_eq(s, x)
            closure(s).contains(x)
            closure(s.c).contains(x)
            closure(s.c.c).contains(x)
            intersection_contains_intro(closure(s.c), closure(s.c.c), x)
            left.contains(x)
        }
        left.contains(x) = right.contains(x)
    }
    set_ext(left, right)
}

/// The boundary is unchanged by taking the complement, in the reverse direction.
theorem boundary_eq_boundary_complement(s: Set[Real]) {
    boundary(s) = boundary(s.c)
} by {
    boundary_complement_eq(s)
}

/// If a set is closed, its boundary is contained in the set.
theorem boundary_subset_of_closed_set(s: Set[Real]) {
    is_closed_set(s) implies boundary(s).subset(s)
} by {
    if is_closed_set(s) {
        boundary_subset_closure(s)
        closure(s).subset(s)
        forall(x: Real) {
            if boundary(s).contains(x) {
                closure(s).contains(x)
                s.contains(x)
            }
        }
    }
}

/// If a set is open, its boundary lies outside the set.
theorem boundary_subset_complement_of_open_set(s: Set[Real]) {
    is_open_set(s) implies boundary(s).subset(s.c)
} by {
    if is_open_set(s) {
        open_set_eq_interior(s)
        s = interior(s)
        boundary_subset_interior_complement(s)
        boundary(s).subset(interior(s).c)
        boundary(s).subset(s.c)
    }
}

/// If a set is open, its boundary is closure minus the set.
theorem boundary_eq_closure_difference_of_open_set(s: Set[Real]) {
    is_open_set(s) implies boundary(s) = closure(s).difference(s)
} by {
    if is_open_set(s) {
        open_set_eq_interior(s)
        s = interior(s)
        boundary_eq_closure_difference_interior(s)
        boundary(s) = closure(s).difference(interior(s))
        boundary(s) = closure(s).difference(s)
    }
}

/// If a set is closed, its boundary is the set minus its interior.
theorem boundary_eq_set_difference_interior_of_closed_set(s: Set[Real]) {
    is_closed_set(s) implies boundary(s) = s.difference(interior(s))
} by {
    if is_closed_set(s) {
        boundary_eq_closure_difference_interior(s)
        boundary(s) = closure(s).difference(interior(s))
        closure(s).subset(s)
        closure(s) = s
        boundary(s) = s.difference(interior(s))
    }
}

/// The boundary is disjoint from the interior.
theorem boundary_intersection_interior_is_empty(s: Set[Real]) {
    boundary(s).intersection(interior(s)) = Set[Real].empty_set
} by {
    let i = boundary(s).intersection(interior(s))
    forall(x: Real) {
        if i.contains(x) {
            intersection_contains_eq(boundary(s), interior(s), x)
            boundary(s).contains(x)
            interior(s).contains(x)
            interior_not_boundary(s, x)
            false
        }
    }
    i.subset(Set[Real].empty_set)
}

/// The boundary is disjoint from the exterior.
theorem boundary_intersection_exterior_is_empty(s: Set[Real]) {
    boundary(s).intersection(exterior(s)) = Set[Real].empty_set
} by {
    let i = boundary(s).intersection(exterior(s))
    forall(x: Real) {
        if i.contains(x) {
            intersection_contains_eq(boundary(s), exterior(s), x)
            boundary(s).contains(x)
            exterior(s).contains(x)
            exterior_not_boundary(s, x)
            false
        }
    }
    i.subset(Set[Real].empty_set)
}

/// The exterior is disjoint from the boundary.
theorem exterior_intersection_boundary_is_empty(s: Set[Real]) {
    exterior(s).intersection(boundary(s)) = Set[Real].empty_set
} by {
    intersection_comm[Real](exterior(s), boundary(s))
    exterior(s).intersection(boundary(s)) = boundary(s).intersection(exterior(s))
    boundary_intersection_exterior_is_empty(s)
}
