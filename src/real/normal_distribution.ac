/// The standard normal distribution.
///
/// The standard normal density is phi(x) = (-x^2 / 2).exp / (2 pi).sqrt.  This
/// file develops:
///   - the normalizing constant (2 pi).sqrt, as the value of the real square
///     root at two pi (the root exists because two pi is positive),
///   - the density function and its strict positivity,
///   - the density is bounded above by 1 / (2 pi).sqrt.
///
/// The Gaussian integral  integral(phi, -inf, inf) = 1  is a hard theorem
/// (it requires improper integrals, the polar-coordinate change of variables,
/// and the evaluation of the gamma integral), and is explicitly out of scope
/// here.  Continuity and integrability of phi on compact intervals also need
/// machinery the library does not yet expose to this package (continuity of
/// the exponential, the mean-value-theorem Lipschitz bridge, and the
/// cross-partition comparison of Darboux sums); they are recorded below as
/// comments with the exact missing ingredients.

from real import Real, exp_pos, exp_zero, sqrt_mul_self, sqrt_value_nonneg, two, two_positive, pi, pi_pos, mul_pos_pos, mul_one_over, gt_zero_imp_pos, pos_gt_zero
from ordered_field import inverse_of_positive_is_positive
from algebra.ring.ring import mul_neg_neg
from order import lt_imp_lte, not_lte_imp_gt
from data.basic.functions import identity_fn

numerals Real

/// Two pi is positive.
lemma two_pi_pos {
    two * pi > Real.0
} by {
    two_positive
    two > Real.0
    pi_pos
    pi > Real.0
    mul_pos_pos(two, pi)
    (two * pi).is_positive
    pos_gt_zero(two * pi)
    two * pi > Real.0
}

/// The square root of two pi exists.
theorem normalizing_const_exists {
    exists(y: Real) {
        (two * pi).sqrt = Option.some(y) and y > Real.0
    }
} by {
    two_pi_pos
    two * pi > Real.0
    lt_imp_lte(Real.0, two * pi)
    Real.0 <= two * pi
    sqrt_mul_self(two * pi)
    let y: Real satisfy {
        (two * pi).sqrt = Option.some(y) and y * y = two * pi
    }
    (two * pi).sqrt = Option.some(y)
    y * y = two * pi
    sqrt_value_nonneg(two * pi, y)
    y >= Real.0
    y * y = two * pi
    two * pi != Real.0
    y * y != Real.0
    if y = Real.0 {
        y * y = Real.0 * Real.0
        Real.0 * Real.0 = Real.0
        y * y = Real.0
        false
    }
    y != Real.0
    Real.0 <= y and Real.0 != y
    not_lte_imp_gt(Real.0, y)
    Real.0 < y
    exists(witness: Real) {
        (two * pi).sqrt = Option.some(witness) and witness > Real.0
    }
}

/// The normalizing constant of the standard normal density: (2 pi).sqrt.
let normalizing_const: Real satisfy {
    (two * pi).sqrt = Option.some(normalizing_const)
}

/// The normalizing constant is positive.
theorem normalizing_const_pos {
    normalizing_const > Real.0
} by {
    two_pi_pos
    two * pi > Real.0
    lt_imp_lte(Real.0, two * pi)
    Real.0 <= two * pi
    sqrt_mul_self(two * pi)
    let y: Real satisfy {
        (two * pi).sqrt = Option.some(y) and y * y = two * pi
    }
    (two * pi).sqrt = Option.some(y)
    (two * pi).sqrt = Option.some(normalizing_const)
    some_injective[Real](y, normalizing_const)
    y = normalizing_const
    y * y = two * pi
    normalizing_const * normalizing_const = two * pi
    sqrt_value_nonneg(two * pi, normalizing_const)
    normalizing_const >= Real.0
    two * pi != Real.0
    normalizing_const * normalizing_const != Real.0
    if normalizing_const = Real.0 {
        normalizing_const * normalizing_const = Real.0 * Real.0
        Real.0 * Real.0 = Real.0
        normalizing_const * normalizing_const = Real.0
        false
    }
    normalizing_const != Real.0
    Real.0 <= normalizing_const and Real.0 != normalizing_const
    not_lte_imp_gt(Real.0, normalizing_const)
    Real.0 < normalizing_const
}

/// The standard normal density: (-x^2 / 2).exp / (2 pi).sqrt.
define normal_pdf(x: Real) -> Real {
    (-(x * x) * Real.one_half).exp * (Real.1 / normalizing_const)
}

/// The reciprocal of the normalizing constant is positive.
lemma normalizing_const_inv_pos {
    Real.1 / normalizing_const > Real.0
} by {
    normalizing_const_pos
    normalizing_const > Real.0
    gt_zero_imp_pos(normalizing_const)
    normalizing_const.is_positive
    inverse_of_positive_is_positive(normalizing_const)
    Real.0 < normalizing_const.inverse
    gt_zero_imp_pos(normalizing_const.inverse)
    normalizing_const.inverse.is_positive
    Real.1.is_positive
    (Real.1 * normalizing_const.inverse).is_positive
    Real.1 / normalizing_const = Real.1 * normalizing_const.inverse
    (Real.1 / normalizing_const).is_positive
    pos_gt_zero(Real.1 / normalizing_const)
    Real.1 / normalizing_const > Real.0
}

/// The normal density is strictly positive.
theorem normal_pdf_pos(x: Real) {
    normal_pdf(x) > Real.0
} by {
    exp_pos(-(x * x) * Real.one_half)
    (-(x * x) * Real.one_half).exp > Real.0
    gt_zero_imp_pos((-(x * x) * Real.one_half).exp)
    (-(x * x) * Real.one_half).exp.is_positive
    normalizing_const_inv_pos
    Real.1 / normalizing_const > Real.0
    gt_zero_imp_pos(Real.1 / normalizing_const)
    (Real.1 / normalizing_const).is_positive
    mul_pos_pos((-(x * x) * Real.one_half).exp, Real.1 / normalizing_const)
    ((-(x * x) * Real.one_half).exp * (Real.1 / normalizing_const)).is_positive
    pos_gt_zero((-(x * x) * Real.one_half).exp * (Real.1 / normalizing_const))
    (-(x * x) * Real.one_half).exp * (Real.1 / normalizing_const) > Real.0
    normal_pdf(x) = (-(x * x) * Real.one_half).exp * (Real.1 / normalizing_const)
    normal_pdf(x) > Real.0
}

/// The normal density is nonnegative.
theorem normal_pdf_nonneg(x: Real) {
    Real.0 <= normal_pdf(x)
} by {
    normal_pdf_pos(x)
    normal_pdf(x) > Real.0
    lt_imp_lte(Real.0, normal_pdf(x))
    Real.0 <= normal_pdf(x)
}

/// The standard normal density is even: φ(-x) = φ(x).
theorem normal_pdf_even(x: Real) {
    normal_pdf(-x) = normal_pdf(x)
} by {
    normal_pdf(-x) = (-((-x) * (-x)) * Real.one_half).exp * (Real.1 / normalizing_const)
    mul_neg_neg(x, x)
    (-x) * (-x) = x * x
    (-((-x) * (-x)) * Real.one_half).exp = (-(x * x) * Real.one_half).exp
    normal_pdf(-x) = (-(x * x) * Real.one_half).exp * (Real.1 / normalizing_const)
    normal_pdf(x) = (-(x * x) * Real.one_half).exp * (Real.1 / normalizing_const)
    normal_pdf(-x) = normal_pdf(x)
}

/// The standard normal density at zero is 1 / (2 pi).sqrt.
theorem normal_pdf_at_zero {
    normal_pdf(Real.0) = Real.1 / normalizing_const
} by {
    normal_pdf(Real.0) = (-(Real.0 * Real.0) * Real.one_half).exp * (Real.1 / normalizing_const)
    Real.0 * Real.0 = Real.0
    Real.0 * Real.one_half = Real.0
    -(Real.0 * Real.0) * Real.one_half = -(Real.0)
    -(Real.0) = Real.0
    -(Real.0 * Real.0) * Real.one_half = Real.0
    (-(Real.0 * Real.0) * Real.one_half).exp = (Real.0).exp
    exp_zero
    (Real.0).exp = Real.1
    normal_pdf(Real.0) = Real.1 * (Real.1 / normalizing_const)
    Real.1 * (Real.1 / normalizing_const) = Real.1 / normalizing_const
    normal_pdf(Real.0) = Real.1 / normalizing_const
}

// ---------------------------------------------------------------------------
// Continuity and integrability of the normal density (future work)
// ---------------------------------------------------------------------------
//
// The standard normal density is continuous on the whole real line: it is the
// composition of the continuous exponential with the continuous quadratic
// -(x^2)/2, scaled by the nonzero constant 1 / (2 pi).sqrt.  A proof needs the
// continuity of Real.exp and the composition rule for continuous functions, which
// the real package does not currently export to this package.
//
// The density is bounded on every compact interval [a, b] between zero and
// 1 / (2 pi).sqrt, and it is Riemann integrable on [a, b].  A proof of
// integrability needs the mean-value-theorem Lipschitz bridge (the derivative
// of phi is bounded in absolute value by 1 / (2 pi).sqrt on the whole line)
// together with the Darboux criterion; the real package exposes the Darboux
// machinery (fn_integrable_gen, darboux_criterion) but not the extreme-value
// theorem and the cross-partition comparison that the bridge requires.
//
// The Gaussian integral  integral(normal_pdf, a, b) -> 1  as a -> -inf,
// b -> inf  needs improper integrals and the evaluation of the Gaussian
// integral itself (integral((-x^2/2).exp) over the line is (2 pi).sqrt), which
// is a hard theorem in its own right and is explicitly out of scope.
