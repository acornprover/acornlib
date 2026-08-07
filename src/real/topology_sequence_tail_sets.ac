from nat import Nat
from data.basic.set import Set, subset_contains, subset_trans
from real.real_field import Real
from real.real_seq import converges_to
from real.sequence_set_membership import seq_eventually_in_real_set, seq_frequently_in_real_set,
    seq_in_real_set
from real.sequence_tail_sets import sequence_range_set, sequence_range_set_contains_sequence,
    sequence_range_set_contains_term, sequence_tail_meets_set,
    sequence_tail_set, sequence_tail_set_contains_term, sequence_tail_set_eventually_contains_sequence,
    sequence_tail_set_subset_of_le, sequence_tail_set_subset_range_set,
    seq_eventually_in_real_set_iff_tail_subset,
    seq_frequently_in_real_set_iff_tail_meets, seq_in_real_set_iff_range_subset
from real.topology import closure, closure_mono, is_closed_set, is_limit_point_of_set,
    point_in_closure_if_in_set
from real.topology_sequence_closure import closed_set_contains_eventual_seq_limit,
    closed_set_contains_frequent_seq_limit, closed_set_contains_seq_in_real_set_limit,
    seq_eventually_converges_imp_closure_contains,
    seq_frequently_converges_imp_closure_contains,
    seq_frequently_in_punctured_set_converges_imp_limit_point,
    seq_in_real_set_converges_imp_closure_contains

/// The closure of the range contains every sequence term.
theorem sequence_range_closure_contains_term(a: Nat -> Real, n: Nat) {
    closure(sequence_range_set(a)).contains(a(n))
} by {
    sequence_range_set_contains_term(a, n)
    sequence_range_set(a).contains(a(n))
    point_in_closure_if_in_set(sequence_range_set(a), a(n))
    closure(sequence_range_set(a)).contains(a(n))
}

/// The closure of a tail contains every sufficiently late sequence term.
theorem sequence_tail_closure_contains_late_term(a: Nat -> Real, m: Nat, n: Nat) {
    m <= n implies closure(sequence_tail_set(a, m)).contains(a(n))
} by {
    if m <= n {
        sequence_tail_set_contains_term(a, m, n)
        sequence_tail_set(a, m).contains(a(n))
        point_in_closure_if_in_set(sequence_tail_set(a, m), a(n))
        closure(sequence_tail_set(a, m)).contains(a(n))
    }
}

/// The closure of a tail is contained in the closure of the range.
theorem sequence_tail_closure_subset_range_closure(a: Nat -> Real, m: Nat) {
    closure(sequence_tail_set(a, m)).subset(closure(sequence_range_set(a)))
} by {
    sequence_tail_set_subset_range_set(a, m)
    sequence_tail_set(a, m).subset(sequence_range_set(a))
    closure_mono(sequence_tail_set(a, m), sequence_range_set(a))
    closure(sequence_tail_set(a, m)).subset(closure(sequence_range_set(a)))
}

/// Later tail closures are contained in earlier tail closures.
theorem sequence_tail_closure_subset_of_le(a: Nat -> Real, m: Nat, n: Nat) {
    m <= n implies closure(sequence_tail_set(a, n)).subset(closure(sequence_tail_set(a, m)))
} by {
    if m <= n {
        sequence_tail_set_subset_of_le(a, m, n)
        sequence_tail_set(a, n).subset(sequence_tail_set(a, m))
        closure_mono(sequence_tail_set(a, n), sequence_tail_set(a, m))
        closure(sequence_tail_set(a, n)).subset(closure(sequence_tail_set(a, m)))
    }
}

/// The limit of a convergent sequence lies in the closure of its range.
theorem convergent_sequence_limit_in_range_closure(a: Nat -> Real, x: Real) {
    converges_to(a, x) implies closure(sequence_range_set(a)).contains(x)
} by {
    if converges_to(a, x) {
        sequence_range_set_contains_sequence(a)
        seq_in_real_set(sequence_range_set(a), a)
        seq_in_real_set_converges_imp_closure_contains(sequence_range_set(a), a, x)
        closure(sequence_range_set(a)).contains(x)
    }
}

/// The limit of a convergent sequence lies in the closure of every tail.
theorem convergent_sequence_limit_in_tail_closure(a: Nat -> Real, m: Nat, x: Real) {
    converges_to(a, x) implies closure(sequence_tail_set(a, m)).contains(x)
} by {
    if converges_to(a, x) {
        sequence_tail_set_eventually_contains_sequence(a, m)
        seq_eventually_in_real_set(sequence_tail_set(a, m), a)
        seq_eventually_converges_imp_closure_contains(sequence_tail_set(a, m), a, x)
        closure(sequence_tail_set(a, m)).contains(x)
    }
}

/// A range contained in a set puts the sequence limit in that set's closure.
theorem convergent_sequence_limit_in_closure_of_range_superset(
    a: Nat -> Real, x: Real, s: Set[Real]
) {
    converges_to(a, x) and sequence_range_set(a).subset(s) implies closure(s).contains(x)
} by {
    if converges_to(a, x) and sequence_range_set(a).subset(s) {
        seq_in_real_set_iff_range_subset(s, a)
        seq_in_real_set(s, a)
        seq_in_real_set_converges_imp_closure_contains(s, a, x)
        closure(s).contains(x)
    }
}

/// A tail contained in a set puts the sequence limit in that set's closure.
theorem convergent_sequence_limit_in_closure_of_tail_superset(
    a: Nat -> Real, m: Nat, x: Real, s: Set[Real]
) {
    converges_to(a, x) and sequence_tail_set(a, m).subset(s) implies closure(s).contains(x)
} by {
    if converges_to(a, x) and sequence_tail_set(a, m).subset(s) {
        seq_eventually_in_real_set_iff_tail_subset(s, a)
        exists(k: Nat) {
            sequence_tail_set(a, k).subset(s)
        }
        seq_eventually_in_real_set(s, a)
        seq_eventually_converges_imp_closure_contains(s, a, x)
        closure(s).contains(x)
    }
}

/// A closed superset of the range contains the sequence limit.
theorem closed_set_contains_convergent_sequence_range_limit(
    a: Nat -> Real, x: Real, s: Set[Real]
) {
    is_closed_set(s) and converges_to(a, x) and sequence_range_set(a).subset(s) implies s.contains(x)
} by {
    if is_closed_set(s) and converges_to(a, x) and sequence_range_set(a).subset(s) {
        seq_in_real_set_iff_range_subset(s, a)
        seq_in_real_set(s, a)
        closed_set_contains_seq_in_real_set_limit(s, a, x)
        s.contains(x)
    }
}

/// A closed superset of a tail contains the sequence limit.
theorem closed_set_contains_convergent_sequence_tail_limit(
    a: Nat -> Real, m: Nat, x: Real, s: Set[Real]
) {
    is_closed_set(s) and converges_to(a, x) and sequence_tail_set(a, m).subset(s) implies s.contains(x)
} by {
    if is_closed_set(s) and converges_to(a, x) and sequence_tail_set(a, m).subset(s) {
        seq_eventually_in_real_set_iff_tail_subset(s, a)
        exists(k: Nat) {
            sequence_tail_set(a, k).subset(s)
        }
        seq_eventually_in_real_set(s, a)
        closed_set_contains_eventual_seq_limit(s, a, x)
        s.contains(x)
    }
}

/// If every tail meets a set, a convergent sequence has its limit in that set's closure.
theorem convergent_sequence_limit_in_closure_of_tail_meeting_set(
    a: Nat -> Real, x: Real, s: Set[Real]
) {
    converges_to(a, x) and forall(m: Nat) { sequence_tail_meets_set(a, m, s) }
    implies closure(s).contains(x)
} by {
    if converges_to(a, x) and forall(m: Nat) { sequence_tail_meets_set(a, m, s) } {
        seq_frequently_in_real_set_iff_tail_meets(s, a)
        seq_frequently_in_real_set(s, a)
        seq_frequently_converges_imp_closure_contains(s, a, x)
        closure(s).contains(x)
    }
}

/// If every tail meets a closed set, a convergent sequence has its limit in that set.
theorem closed_set_contains_convergent_sequence_tail_meeting_limit(
    a: Nat -> Real, x: Real, s: Set[Real]
) {
    is_closed_set(s) and converges_to(a, x) and forall(m: Nat) { sequence_tail_meets_set(a, m, s) }
    implies s.contains(x)
} by {
    if is_closed_set(s) and converges_to(a, x) and forall(m: Nat) { sequence_tail_meets_set(a, m, s) } {
        seq_frequently_in_real_set_iff_tail_meets(s, a)
        seq_frequently_in_real_set(s, a)
        closed_set_contains_frequent_seq_limit(s, a, x)
        s.contains(x)
    }
}

/// If every tail meets a punctured set, a convergent sequence gives a limit point.
theorem convergent_sequence_tail_meets_punctured_set_imp_limit_point(
    a: Nat -> Real, x: Real, s: Set[Real]
) {
    converges_to(a, x) and
    forall(m: Nat) { sequence_tail_meets_set(a, m, s.difference(Set[Real].singleton(x))) }
    implies is_limit_point_of_set(s, x)
} by {
    if converges_to(a, x) and
       forall(m: Nat) { sequence_tail_meets_set(a, m, s.difference(Set[Real].singleton(x))) } {
        seq_frequently_in_real_set_iff_tail_meets(s.difference(Set[Real].singleton(x)), a)
        seq_frequently_in_real_set(s.difference(Set[Real].singleton(x)), a)
        seq_frequently_in_punctured_set_converges_imp_limit_point(s, a, x)
        is_limit_point_of_set(s, x)
    }
}

/// Tail closure containment transports to range closure containment.
theorem tail_closure_member_is_range_closure_member(a: Nat -> Real, m: Nat, x: Real) {
    closure(sequence_tail_set(a, m)).contains(x) implies closure(sequence_range_set(a)).contains(x)
} by {
    if closure(sequence_tail_set(a, m)).contains(x) {
        sequence_tail_closure_subset_range_closure(a, m)
        closure(sequence_tail_set(a, m)).subset(closure(sequence_range_set(a)))
        subset_contains(closure(sequence_tail_set(a, m)), closure(sequence_range_set(a)), x)
        closure(sequence_range_set(a)).contains(x)
    }
}

/// Later tail closure membership transports to earlier tail closure membership.
theorem tail_closure_member_of_le(a: Nat -> Real, m: Nat, n: Nat, x: Real) {
    m <= n and closure(sequence_tail_set(a, n)).contains(x)
    implies closure(sequence_tail_set(a, m)).contains(x)
} by {
    if m <= n and closure(sequence_tail_set(a, n)).contains(x) {
        sequence_tail_closure_subset_of_le(a, m, n)
        closure(sequence_tail_set(a, n)).subset(closure(sequence_tail_set(a, m)))
        subset_contains(closure(sequence_tail_set(a, n)), closure(sequence_tail_set(a, m)), x)
        closure(sequence_tail_set(a, m)).contains(x)
    }
}

/// Closure of a later tail is contained in the closure of the range through transitivity.
theorem tail_closure_subset_range_closure_of_later_tail(a: Nat -> Real, m: Nat, n: Nat) {
    m <= n implies closure(sequence_tail_set(a, n)).subset(closure(sequence_range_set(a)))
} by {
    if m <= n {
        sequence_tail_closure_subset_of_le(a, m, n)
        closure(sequence_tail_set(a, n)).subset(closure(sequence_tail_set(a, m)))
        sequence_tail_closure_subset_range_closure(a, m)
        closure(sequence_tail_set(a, m)).subset(closure(sequence_range_set(a)))
        subset_trans[Real](closure(sequence_tail_set(a, n)), closure(sequence_tail_set(a, m)), closure(sequence_range_set(a)))
        closure(sequence_tail_set(a, n)).subset(closure(sequence_range_set(a)))
    }
}
