from data.basic.set import Set, difference_contains_intro, double_inclusion, singleton_contains_eq,
    sets_subset_contain_union, union_contains_left, union_contains_right
from real.real_field import Real
from real.topology import eps_adherent_of_contains_close, is_adherent_point_of_set,
    is_closed_set, is_eps_adherent_to_set, is_isolated_point_of_set,
    is_limit_point_of_set
from real.topology_complements import not_adherent_has_missing_eps
from real.topology_derived_set import derived_set, derived_set_contains_eq,
    derived_set_subset_of_closed_set
from real.topology_isolated_set import isolated_set, isolated_set_contains_eq,
    isolated_set_subset_self
from real.topology_perfect_sets import is_perfect_real_set, perfect_real_set_intro

/// A point of a real set that is not in the derived set is isolated.
theorem member_not_in_derived_set_is_isolated_point(s: Set[Real], x: Real) {
    s.contains(x) and not derived_set(s).contains(x) implies is_isolated_point_of_set(s, x)
} by {
    if s.contains(x) and not derived_set(s).contains(x) {
        if is_limit_point_of_set(s, x) {
            derived_set_contains_eq(s, x)
            derived_set(s).contains(x)
            false
        }
        not is_limit_point_of_set(s, x)
        is_limit_point_of_set(s, x) = is_adherent_point_of_set(s.difference(Set[Real].singleton(x)), x)
        not is_adherent_point_of_set(s.difference(Set[Real].singleton(x)), x)
        not_adherent_has_missing_eps(s.difference(Set[Real].singleton(x)), x)
        let eps: Real satisfy {
            eps.is_positive and not is_eps_adherent_to_set(s.difference(Set[Real].singleton(x)), x, eps)
        }
        forall(y: Real) {
            if s.contains(y) and y != x {
                if y.is_close(x, eps) {
                    if Set[Real].singleton(x).contains(y) {
                        singleton_contains_eq[Real](x, y)
                        y = x
                        false
                    }
                    not Set[Real].singleton(x).contains(y)
                    difference_contains_intro[Real](s, Set[Real].singleton(x), y)
                    s.difference(Set[Real].singleton(x)).contains(y)
                    eps_adherent_of_contains_close(s.difference(Set[Real].singleton(x)), x, y, eps)
                    is_eps_adherent_to_set(s.difference(Set[Real].singleton(x)), x, eps)
                    false
                }
                not y.is_close(x, eps)
            }
        }
        exists(delta: Real) {
            delta.is_positive and forall(y: Real) {
                s.contains(y) and y != x implies not y.is_close(x, delta)
            }
        }
        is_isolated_point_of_set(s, x)
    }
}

/// A point of a real set outside the derived set lies in the isolated-point set.
theorem member_not_in_derived_set_is_isolated_set_member(s: Set[Real], x: Real) {
    s.contains(x) and not derived_set(s).contains(x) implies isolated_set(s).contains(x)
} by {
    if s.contains(x) and not derived_set(s).contains(x) {
        member_not_in_derived_set_is_isolated_point(s, x)
        is_isolated_point_of_set(s, x)
        isolated_set_contains_eq(s, x)
        isolated_set(s).contains(x)
    }
}

/// Every point of a real set is either a limit point of the set or isolated in it.
theorem subset_derived_union_isolated_set(s: Set[Real]) {
    s.subset(derived_set(s).union(isolated_set(s)))
} by {
    forall(x: Real) {
        if s.contains(x) {
            if derived_set(s).contains(x) {
                union_contains_left(derived_set(s), isolated_set(s), x)
                derived_set(s).union(isolated_set(s)).contains(x)
            } else {
                member_not_in_derived_set_is_isolated_set_member(s, x)
                isolated_set(s).contains(x)
                union_contains_right(derived_set(s), isolated_set(s), x)
                derived_set(s).union(isolated_set(s)).contains(x)
            }
        }
    }
}

/// First Cantor--Bendixson bridge: a closed real set decomposes into its
/// derived set together with its isolated-point set.
theorem closed_set_eq_derived_union_isolated_set(s: Set[Real]) {
    is_closed_set(s) implies s = derived_set(s).union(isolated_set(s))
} by {
    if is_closed_set(s) {
        subset_derived_union_isolated_set(s)
        s.subset(derived_set(s).union(isolated_set(s)))
        derived_set_subset_of_closed_set(s)
        derived_set(s).subset(s)
        isolated_set_subset_self(s)
        isolated_set(s).subset(s)
        sets_subset_contain_union[Real](derived_set(s), isolated_set(s), s)
        derived_set(s).union(isolated_set(s)).subset(s)
        double_inclusion[Real](s, derived_set(s).union(isolated_set(s)))
        s = derived_set(s).union(isolated_set(s))
    }
}

/// A closed real set with no isolated points is perfect.
theorem closed_set_with_empty_isolated_set_is_perfect(s: Set[Real]) {
    is_closed_set(s) and isolated_set(s).is_empty implies is_perfect_real_set(s)
} by {
    if is_closed_set(s) and isolated_set(s).is_empty {
        forall(x: Real) {
            if s.contains(x) {
                if derived_set(s).contains(x) {
                } else {
                    member_not_in_derived_set_is_isolated_set_member(s, x)
                    isolated_set(s).contains(x)
                    false
                }
            }
        }
        s.subset(derived_set(s))
        perfect_real_set_intro(s)
        is_perfect_real_set(s)
    }
}
