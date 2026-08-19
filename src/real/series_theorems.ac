/// Canonical series theorems: the geometric series, the harmonic series, the
/// p-series, the ratio test, and the alternating series test.
///
/// Each theorem restates a result proved elsewhere in the library (the
/// geometric-series value in `real.real_field`, the zeta(2) convergence in
/// `real.zeta_values`, the ratio and alternating tests in `real.series_tests`)
/// under the conventional statement of the result.

from nat import Nat, from_nat, from_nat_add, from_nat_one, from_nat_zero, lt_suc, lt_not_ref, lt_and_lte
from list import partial
from algebra.field.field import unique_inverse
from order import lte_trans, lt_iff_lte_and_ne
from real.real_base import Real, close_comm, close_imp_bounds, one_half_plus_one_half, lte_add_right, lte_add_left
from real.real_ring import converges, limit, converges_to, real_mul_comm
from real.real_seq import cauchy_bound
from real.real_field import geom_series
from real.real_series import geom_converges
from real.harmonic import harmonic, harmonic_block_term_lower
from real.zeta_values import zeta2_term, zeta2_converges, zeta2_partial_le_two
from real.series_tests import ratio_test, alternating_series_test, alt_term, ratio_seq, is_decreasing_seq
from real.abs_conv import absolutely_converges, absolutely_converges_imp_converges
from real.exp import always_nonzero, mul_one_over
from real.limits import vanishes

numerals Real

// ---------------------------------------------------------------------------
// (a) The geometric series: Σ_{n=0}^∞ r^n = 1 / (1 - r) for |r| < 1.

/// The geometric series Σ r^n converges when |r| < 1.
theorem geometric_series_converges(r: Real) {
    r.abs < Real.1 implies converges(partial(r.pow))
} by {
    if r.abs < Real.1 {
        geom_converges(r)
        converges(partial(r.pow))
    }
}

/// The geometric series sums to 1 / (1 - r) when |r| < 1.
theorem geometric_series_sum(r: Real) {
    r.abs < Real.1 implies limit(partial(r.pow)) = Real.1 / (Real.1 - r)
} by {
    if r.abs < Real.1 {
        geom_series(r)
        limit(partial(r.pow)) = Real.1 / (Real.1 - r)
    }
}

// ---------------------------------------------------------------------------
// (c) The p-series Σ 1 / n² converges (the zeta(2) series).

/// The p-series Σ 1/n² converges.
theorem p_series_two_converges {
    converges(partial(zeta2_term))
} by {
    zeta2_converges
    converges(partial(zeta2_term))
}

/// The partial sums of the p-series Σ 1/n² are bounded by two.
theorem p_series_two_partials_le_two(n: Nat) {
    partial(zeta2_term, n) <= Real.1 + Real.1
} by {
    zeta2_partial_le_two(n)
    partial(zeta2_term, n) <= Real.1 + Real.1
}

// ---------------------------------------------------------------------------
// (d) The ratio test: if |a(n+1)| / |a(n)| converges to l < 1, then Σ a(n)
// converges (absolutely).

/// The ratio test: convergence of the ratio sequence to l < 1 gives absolute
/// convergence of the series.
theorem ratio_test_series_abs_converges(a: Nat -> Real, l: Real) {
    converges_to(ratio_seq(a), l) and l < Real.1 and always_nonzero(a)
    implies absolutely_converges(a)
} by {
    if converges_to(ratio_seq(a), l) and l < Real.1 and always_nonzero(a) {
        ratio_test(a, l)
        absolutely_converges(a)
    }
}

/// The ratio test: convergence of the ratio sequence to l < 1 gives
/// convergence of the series.
theorem ratio_test_series_converges(a: Nat -> Real, l: Real) {
    converges_to(ratio_seq(a), l) and l < Real.1 and always_nonzero(a)
    implies converges(partial(a))
} by {
    if converges_to(ratio_seq(a), l) and l < Real.1 and always_nonzero(a) {
        ratio_test(a, l)
        absolutely_converges(a)
        absolutely_converges_imp_converges(a)
        converges(partial(a))
    }
}

// ---------------------------------------------------------------------------
// (e) The alternating series test (Leibniz): if a decreases to zero, then
// Σ (-1)^n a(n) converges.

/// The alternating series test: a nonnegative sequence decreasing to zero
/// gives a convergent alternating series.
theorem alternating_series_test_series_converges(a: Nat -> Real) {
    is_decreasing_seq(a) and vanishes(a) implies converges(partial(alt_term(a)))
} by {
    if is_decreasing_seq(a) and vanishes(a) {
        alternating_series_test(a)
        converges(partial(alt_term(a)))
    }
}

// ---------------------------------------------------------------------------
// (b) The harmonic series Σ 1 / n diverges, by the grouping argument:
// 1 + 1/2 + (1/3 + 1/4) + ... has each dyadic block at least one half, so the
// partial sums at 2^k grow by at least 1/2 per block; a convergent sequence
// would be Cauchy at radius 1/2, contradicting the half-block jumps.

/// Adding two lower bounds gives a lower bound on the sum.
theorem real_add_two_lower(a: Real, b: Real, u: Real, v: Real) {
    a >= u and b >= v implies a + b >= u + v
} by {
    if a >= u and b >= v {
        lte_add_right(u, a, b)
        u + b <= a + b
        lte_add_left(v, b, u)
        u + v <= u + b
        lte_trans(u + v, u + b, a + b)
        u + v <= a + b
        a + b >= u + v
    }
}

/// Multiplication by a reciprocal respects successor expansion of natural reals.
theorem from_nat_suc_mul_recip_step(x: Nat, d: Nat) {
    from_nat[Real](x.suc) * (Real.1 / from_nat[Real](d)) = from_nat[Real](x) * (Real.1 / from_nat[Real](d)) + (Real.1 / from_nat[Real](d))
} by {
    mul_one_over(Real.1, Real.1)
    x.suc = x + Nat.1
    from_nat[Real](x.suc) = from_nat[Real](x + Nat.1)
    from_nat[Real](x + Nat.1) = from_nat[Real](x) + from_nat[Real](Nat.1)
    from_nat[Real](Nat.1) = Real.1
    from_nat[Real](x.suc) = from_nat[Real](x) + Real.1
    from_nat[Real](x.suc) * (Real.1 / from_nat[Real](d)) = (from_nat[Real](x) + Real.1) * (Real.1 / from_nat[Real](d))
    (from_nat[Real](x) + Real.1) * (Real.1 / from_nat[Real](d)) = from_nat[Real](x) * (Real.1 / from_nat[Real](d)) + Real.1 * (Real.1 / from_nat[Real](d))
    Real.1 * (Real.1 / from_nat[Real](d)) = Real.1 / from_nat[Real](d)
}

/// A partial sum with each term above a common reciprocal grows by that reciprocal.
theorem partial_real_lower_step_generic(f: Nat -> Real, x: Nat, d: Nat) {
    partial(f, x) >= from_nat[Real](x) * (Real.1 / from_nat[Real](d)) and
    f(x) >= Real.1 / from_nat[Real](d) implies
    partial(f, x.suc) >= from_nat[Real](x.suc) * (Real.1 / from_nat[Real](d))
} by {
    partial(f, x.suc) = partial(f, x) + f(x)
    real_add_two_lower(partial(f, x), f(x), from_nat[Real](x) * (Real.1 / from_nat[Real](d)), Real.1 / from_nat[Real](d))
    partial(f, x) + f(x) >= from_nat[Real](x) * (Real.1 / from_nat[Real](d)) + (Real.1 / from_nat[Real](d))
    from_nat_suc_mul_recip_step(x, d)
    from_nat[Real](x.suc) * (Real.1 / from_nat[Real](d)) = from_nat[Real](x) * (Real.1 / from_nat[Real](d)) + (Real.1 / from_nat[Real](d))
}

/// A successor comparison gives the strict order.
theorem lte_suc_imp_lt(a: Nat, b: Nat) {
    a.suc <= b implies a < b
} by {
    if a.suc <= b {
        a <= a.suc
        lte_trans(a, a.suc, b)
        a <= b
        if a = b {
            a.suc <= a
            lt_suc(a)
            a < a.suc
            lt_and_lte(a, a.suc, a)
            a < a
            lt_not_ref(a)
            false
        }
        a != b
        lt_iff_lte_and_ne[Nat](a, b)
        a < b = (a <= b and a != b)
        a < b
    }
}

/// Extending a harmonic block by one preserves the reciprocal block lower bound.
theorem harmonic_block_partial_lower_step(n: Nat, x: Nat) {
    n > Nat.0 and x.suc <= n and
    partial(function(k: Nat) { harmonic(n + k) }, x) >= from_nat[Real](x) * (Real.1 / from_nat[Real](n + n)) implies
    partial(function(k: Nat) { harmonic(n + k) }, x.suc) >= from_nat[Real](x.suc) * (Real.1 / from_nat[Real](n + n))
} by {
    lte_suc_imp_lt(x, n)
    x < n
    harmonic_block_term_lower(n, x)
    partial_real_lower_step_generic(function(k: Nat) { harmonic(n + k) }, x, n + n)
}

/// The first `m` terms of the dyadic harmonic block are bounded below by `m / (2n)`.
theorem harmonic_block_partial_lower(n: Nat, m: Nat) {
    n > Nat.0 and m <= n implies
    partial(function(k: Nat) { harmonic(n + k) }, m) >= from_nat[Real](m) * (Real.1 / from_nat[Real](n + n))
} by {
    define p(x: Nat) -> Bool {
        n > Nat.0 and x <= n implies partial(function(k: Nat) { harmonic(n + k) }, x) >= from_nat[Real](x) * (Real.1 / from_nat[Real](n + n))
    }
    partial(function(k: Nat) { harmonic(n + k) }, Nat.0) = Real.0
    from_nat_zero[Real]
    from_nat[Real](Nat.0) = Real.0
    from_nat[Real](Nat.0) * (Real.1 / from_nat[Real](n + n)) = Real.0
    p(Nat.0)
    forall(x: Nat) {
        if p(x) {
            if n > Nat.0 and x.suc <= n {
                x <= x.suc
                lte_trans(x, x.suc, n)
                x <= n
                partial(function(k: Nat) { harmonic(n + k) }, x) >= from_nat[Real](x) * (Real.1 / from_nat[Real](n + n))
                harmonic_block_partial_lower_step(n, x)
                partial(function(k: Nat) { harmonic(n + k) }, x.suc) >= from_nat[Real](x.suc) * (Real.1 / from_nat[Real](n + n))
            }
            p(x.suc)
        }
    }
    p(Nat.0) and forall(x: Nat) { p(x) implies p(x.suc) }
    Nat.induction(p)
    p(m)
}

/// Twice one half is one.
theorem two_mul_one_half_real {
    (Real.1 + Real.1) * Real.one_half = Real.1
} by {
    real_mul_comm(Real.one_half, Real.1)
    (Real.1 + Real.1) * Real.one_half = Real.1 * Real.one_half + Real.1 * Real.one_half
    Real.1 * Real.one_half = Real.one_half
    (Real.1 + Real.1) * Real.one_half = Real.one_half + Real.one_half
    one_half_plus_one_half
}

/// One half is the inverse of two.
theorem half_is_inverse_two {
    Real.one_half = (Real.1 + Real.1).inverse
} by {
    (Real.1 + Real.1) * Real.one_half = Real.1
    unique_inverse[Real](Real.1 + Real.1, Real.one_half)
}

/// The reciprocal of two is one half.
theorem real_one_div_two_eq_half {
    Real.1 / (Real.1 + Real.1) = Real.one_half
} by {
    half_is_inverse_two
    Real.1 / (Real.1 + Real.1) = Real.1 * (Real.1 + Real.1).inverse
    Real.1 * (Real.1 + Real.1).inverse = (Real.1 + Real.1).inverse
}

/// A nonzero real divided by twice itself is one half.
theorem real_half_ratio_nonzero(a: Real) {
    a != Real.0 implies a * (Real.1 / ((Real.1 + Real.1) * a)) = Real.one_half
} by {
    let two: Real = Real.1 + Real.1
    two != Real.0
    two * a != Real.0
    a * (Real.1 / (two * a)) = a / (two * a)
    a / (two * a) = Real.1 / two
    Real.1 / two = Real.one_half
}

/// A positive successor natural is one half of its double as a reciprocal product.
theorem from_nat_suc_half_ratio(k: Nat) {
    from_nat[Real](k.suc) * (Real.1 / from_nat[Real](k.suc + k.suc)) = Real.one_half
} by {
    let a: Real = from_nat[Real](k.suc)
    from_nat[Real](k.suc + k.suc) = from_nat[Real](k.suc) + from_nat[Real](k.suc)
    from_nat[Real](k.suc + k.suc) = a + a
    a + a = (Real.1 + Real.1) * a
    from_nat[Real](k.suc + k.suc) = (Real.1 + Real.1) * a
    a != Real.0
    a * (Real.1 / ((Real.1 + Real.1) * a)) = Real.one_half
    from_nat[Real](k.suc) * (Real.1 / from_nat[Real](k.suc + k.suc)) = Real.one_half
}

/// A positive natural is nonzero.
theorem nat_pos_ne_zero_atomic(n: Nat) {
    n > Nat.0 implies n != Nat.0
}

/// A nonzero natural real is one half of its double as a reciprocal product.
theorem from_nat_half_ratio_ne_zero(n: Nat) {
    n != Nat.0 implies from_nat[Real](n) * (Real.1 / from_nat[Real](n + n)) = Real.one_half
} by {
    define p(x: Nat) -> Bool {
        x != Nat.0 implies from_nat[Real](x) * (Real.1 / from_nat[Real](x + x)) = Real.one_half
    }
    if Nat.0 != Nat.0 {
        false
    }
    p(Nat.0)
    forall(k: Nat) {
        if p(k) {
            from_nat_suc_half_ratio(k)
            if k.suc != Nat.0 {
                from_nat[Real](k.suc) * (Real.1 / from_nat[Real](k.suc + k.suc)) = Real.one_half
            }
            p(k.suc)
        }
    }
    p(Nat.0) and forall(k: Nat) { p(k) implies p(k.suc) }
    Nat.induction(p)
    p(n)
}

/// A positive natural real is one half of its double as a reciprocal product.
theorem from_nat_half_ratio(n: Nat) {
    n > Nat.0 implies from_nat[Real](n) * (Real.1 / from_nat[Real](n + n)) = Real.one_half
} by {
    nat_pos_ne_zero_atomic(n)
    from_nat_half_ratio_ne_zero(n)
}

/// A full dyadic harmonic block has total at least one half.
theorem harmonic_block_lower(n: Nat) {
    n > Nat.0 implies partial(function(k: Nat) { harmonic(n + k) }, n) >= Real.one_half
} by {
    harmonic_block_partial_lower(n, n)
    from_nat_half_ratio(n)
}

/// Splitting a real partial sum into its first `n` terms and the following `m` terms.
theorem partial_add_shift_real(f: Nat -> Real, n: Nat, m: Nat) {
    partial(f, n + m) = partial(f, n) + partial(function(k: Nat) { f(n + k) }, m)
} by {
    define g(k: Nat) -> Real { f(n + k) }
    define p(x: Nat) -> Bool {
        partial(f, n + x) = partial(f, n) + partial(g, x)
    }
    partial(g, Nat.0) = Real.0
    partial(f, n + Nat.0) = partial(f, n)
    p(Nat.0)
    forall(x: Nat) {
        if p(x) {
            n + x.suc = (n + x).suc
            partial(f, n + x.suc) = partial(f, (n + x).suc)
            partial(f, (n + x).suc) = partial(f, n + x) + f(n + x)
            partial(f, n + x.suc) = partial(f, n + x) + f(n + x)
            partial(f, n + x) + f(n + x) = (partial(f, n) + partial(g, x)) + f(n + x)
            (partial(f, n) + partial(g, x)) + f(n + x) = partial(f, n) + (partial(g, x) + f(n + x))
            partial(g, x.suc) = partial(g, x) + g(x)
            g(x) = f(n + x)
            partial(g, x.suc) = partial(g, x) + f(n + x)
            partial(f, n) + (partial(g, x) + f(n + x)) = partial(f, n) + partial(g, x.suc)
            p(x.suc)
        }
    }
    p(Nat.0) and forall(x: Nat) { p(x) implies p(x.suc) }
    Nat.induction(p)
    p(m)
    forall(k: Nat) { g(k) = f(n + k) }
    partial(g, m) = partial(function(k: Nat) { f(n + k) }, m)
}

/// Doubling any positive partial-sum index increases the harmonic partial sum by at least one half.
theorem harmonic_tail_half_jump(k: Nat) {
    k > Nat.0 implies partial(harmonic, k + k) >= partial(harmonic, k) + Real.one_half
} by {
    partial_add_shift_real(harmonic, k, k)
    partial(harmonic, k + k) = partial(harmonic, k) + partial(function(t: Nat) { harmonic(k + t) }, k)
    harmonic_block_lower(k)
    partial(function(t: Nat) { harmonic(k + t) }, k) >= Real.one_half
    partial(harmonic, k) + partial(function(t: Nat) { harmonic(k + t) }, k) >= partial(harmonic, k) + Real.one_half
}

/// One half is positive.
theorem real_one_half_is_positive {
    Real.one_half.is_positive
} by {
    Real.0 < Real.one_half
}

/// A convergent real sequence has a Cauchy bound at radius one half.
theorem converges_has_half_cauchy_bound(q: Nat -> Real) {
    converges(q) implies exists(n: Nat) { cauchy_bound(q, n, Real.one_half) }
} by {
    if converges(q) {
        converges(q) = forall(eps: Real) {
            eps.is_positive implies exists(n: Nat) { cauchy_bound(q, n, eps) }
        }
        forall(eps: Real) {
            eps.is_positive implies exists(n: Nat) { cauchy_bound(q, n, eps) }
        }
        Real.one_half.is_positive
        exists(n: Nat) { cauchy_bound(q, n, Real.one_half) }
    }
}

/// A Cauchy bound controls every pair of indices beyond the bound.
theorem cauchy_bound_all_indices(q: Nat -> Real, n: Nat, eps: Real) {
    cauchy_bound(q, n, eps) implies forall(i: Nat, j: Nat) {
        n <= i and n <= j implies q(i).is_close(q(j), eps)
    }
}

/// A Cauchy bound controls a chosen pair of indices beyond the bound.
theorem cauchy_bound_indices(q: Nat -> Real, n: Nat, eps: Real, i: Nat, j: Nat) {
    cauchy_bound(q, n, eps) and n <= i and n <= j implies q(i).is_close(q(j), eps)
} by {
    cauchy_bound_all_indices(q, n, eps)
    forall(a: Nat, b: Nat) {
        n <= a and n <= b implies q(a).is_close(q(b), eps)
    }
    q(i).is_close(q(j), eps)
}

/// A gap of at least one half is not one-half-close.
theorem not_close_if_ge_half(a: Real, b: Real) {
    b >= a + Real.one_half implies not b.is_close(a, Real.one_half)
} by {
    if b.is_close(a, Real.one_half) {
        close_imp_bounds(b, a, Real.one_half)
        b < a + Real.one_half
        false
    }
}

/// The harmonic series Σ 1 / n diverges.
theorem harmonic_series_diverges {
    not converges(partial(harmonic))
} by {
    if converges(partial(harmonic)) {
        Real.one_half.is_positive
        converges_has_half_cauchy_bound(partial(harmonic))
        let n: Nat satisfy {
            cauchy_bound(partial(harmonic), n, Real.one_half)
        }
        let k: Nat = n.suc
        k > Nat.0
        n <= k
        k <= k + k
        lte_trans(n, k, k + k)
        n <= k + k
        harmonic_tail_half_jump(k)
        partial(harmonic, k + k) >= partial(harmonic, k) + Real.one_half
        not_close_if_ge_half(partial(harmonic, k), partial(harmonic, k + k))
        cauchy_bound_indices(partial(harmonic), n, Real.one_half, k, k + k)
        partial(harmonic, k).is_close(partial(harmonic, k + k), Real.one_half)
        close_comm(partial(harmonic, k), partial(harmonic, k + k), Real.one_half)
        partial(harmonic, k + k).is_close(partial(harmonic, k), Real.one_half)
        false
    }
}

