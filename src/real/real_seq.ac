/// This file defines Cauchy sequences and proves various things about them.
from nat import Nat, lte_trans
from rat import Rat, iop
from rat import finite_seq_abs_bounded
from rat import no_greatest
from data.basic.functions import compose
from real.real_base import Real, is_cut, is_lower, is_greatest, has_greatest, is_dedekind_cut, add_rat_eps_between, close_imp_bounds, close_comm, pos_gt_zero, gt_zero_imp_pos, rat_between_reals, rat_between_reals_gt, rat_intersect, lt_some_rat, from_rat_maintains_lte

attributes Real {
}

/// The bound on the end of a sequence that's part of the Cauchy convergence condition.
/// Separating it out like this makes convergence a bit clearer to prove.
define cauchy_bound(q: Nat -> Real, n: Nat, eps: Real) -> Bool {
    forall(i: Nat, j: Nat) {
        n <= i and n <= j implies q(i).is_close(q(j), eps)
    }
}

/// The Cauchy condition for the convergence of a sequence.
define converges(q: Nat -> Real) -> Bool {
    forall(eps: Real) {
        eps.is_positive implies exists(n: Nat) {
            cauchy_bound(q, n, eps)
        }
    }
}

/// lb is eventually a lower bound of this sequence.
define eventual_lb(q: Nat -> Real, lb: Real) -> Bool {
    exists(n: Nat) {
        forall(i: Nat) {
            n <= i implies lb <= q(i)
        }
    }
}

theorem eventual_lb_gte(q: Nat -> Real, lb: Real) {
    eventual_lb(q, lb) implies exists(n: Nat) {
        forall(i: Nat) {
            n <= i implies q(i) >= lb
        }
    }
} by {
    let n: Nat satisfy {
        forall(i: Nat) {
            n <= i implies lb <= q(i)
        }
    }
    forall(i: Nat) {
        if n <= i {
            lb <= q(i)
            q(i) >= lb
        }
    }
}

/// For demonstrating the equivalent of Cauchy sequences and Dedekind cuts.
define cauchy_gt_rat(q: Nat -> Real, r: Rat) -> Bool {
    exists(lb: Real) {
        lb > Real.from_rat(r) and eventual_lb(q, lb)
    }
}

theorem cauchy_gt_exists_lb(q: Nat -> Real, r: Rat) {
    cauchy_gt_rat(q, r) implies exists(lb: Real) {
        Real.from_rat(r) < lb and eventual_lb(q, lb)
    }
} by {
    if cauchy_gt_rat(q, r) {
        let lb: Real satisfy {
            lb > Real.from_rat(r) and eventual_lb(q, lb)
        }
        lb > Real.from_rat(r)
        Real.from_rat(r) < lb
        exists(lb2: Real) {
            Real.from_rat(r) < lb2 and eventual_lb(q, lb2)
        }
    }
}

theorem cauchy_gt_something(q: Nat -> Real) {
    converges(q) implies exists(r: Rat) {
        cauchy_gt_rat(q, r)
    }
} by {
    let eps = Real.1
    Real.from_rat(Rat.1).is_positive
    eps.is_positive
    let n: Nat satisfy {
        cauchy_bound(q, n, eps)
    }
    let lb: Real satisfy {
        lb < q(n) - eps
    }
    let r: Rat satisfy {
        lb > Real.from_rat(r)
    }
    forall(i: Nat) {
        if n <= i {
            lb <= q(i)
        }
    }
    eventual_lb(q, lb)
    cauchy_gt_rat(q, r)
}

theorem cauchy_gt_not_everything(q: Nat -> Real) {
    converges(q) implies exists(r: Rat) {
        not cauchy_gt_rat(q, r)
    }
} by {
    let eps = Real.1
    Real.from_rat(Rat.1).is_positive
    eps.is_positive
    let n: Nat satisfy {
        cauchy_bound(q, n, eps)
    }
    let r: Rat satisfy {
        Real.from_rat(r) > q(n) + eps
    }
    if cauchy_gt_rat(q, r) {
        let lb: Real satisfy {
            lb > Real.from_rat(r) and eventual_lb(q, lb)
        }
        exists(n2: Nat) {
            forall(i: Nat) {
                n2 <= i implies q(i) >= lb
            }
        }
        let n2: Nat satisfy {
            forall(i: Nat) {
                n2 <= i implies q(i) >= lb
            }
        }
        if n <= n2 {
            n <= n
            n2 <= n2
            q(n2) >= lb
            lb <= q(n2)
            q(n2).is_close(q(n), Real.1)
            q(n2) < q(n) + Real.1
            lb < q(n) + Real.1
            not lb < q(n) + Real.1
            false
        } else {
            Real.from_rat(r) < lb + Real.1
            Real.from_rat(r) < q(n)
            false
        }
    }
}

theorem cauchy_gt_is_cut(q: Nat -> Real) {
    converges(q) implies is_cut(cauchy_gt_rat(q))
}

theorem cauchy_gt_lower_step(q: Nat -> Real, r1: Rat, r2: Rat) {
    r1 < r2 and cauchy_gt_rat(q, r2) implies cauchy_gt_rat(q, r1)
} by {
    let lb: Real satisfy {
        lb > Real.from_rat(r2) and eventual_lb(q, lb)
    }
    lb.gt_rat(r2)
    lb.gt_rat(r1)
    lb > Real.from_rat(r1)
    lb > Real.from_rat(r1) and eventual_lb(q, lb)
    exists(lb2: Real) {
        lb2 > Real.from_rat(r1) and eventual_lb(q, lb2)
    }
    cauchy_gt_rat(q, r1)
}

theorem cauchy_gt_lower_step_ordered(q: Nat -> Real, r1: Rat, r2: Rat) {
    cauchy_gt_rat(q, r2) and r1 < r2 implies cauchy_gt_rat(q, r1)
} by {
    if cauchy_gt_rat(q, r2) and r1 < r2 {
    }
}

theorem cauchy_gt_is_lower(q: Nat -> Real) {
    converges(q) implies is_lower(cauchy_gt_rat(q))
} by {
    if converges(q) {
        forall(r1: Rat, r2: Rat) {
            if cauchy_gt_rat(q, r2) and r1 < r2 {
                let lb: Real satisfy {
                    lb > Real.from_rat(r2) and eventual_lb(q, lb)
                }
                lb.gt_rat(r2)
                lb.gt_rat(r1)
                lb > Real.from_rat(r1)
                exists(lb2: Real) {
                    lb2 > Real.from_rat(r1) and eventual_lb(q, lb2)
                }
                cauchy_gt_rat(q, r1)
            }
        }
    }
}

theorem cauchy_gt_has_no_greatest(q: Nat -> Real) {
    converges(q) implies not has_greatest(cauchy_gt_rat(q))
} by {
    if has_greatest(cauchy_gt_rat(q)) {
        let r: Rat satisfy {
            is_greatest(cauchy_gt_rat(q), r)
        }
        cauchy_gt_rat(q, r)
        exists(lb: Real) {
            Real.from_rat(r) < lb and eventual_lb(q, lb)
        }
        let lb: Real satisfy {
            Real.from_rat(r) < lb and eventual_lb(q, lb)
        }
        let eps: Rat satisfy {
            eps.is_positive and Real.from_rat(r) + Real.from_rat(eps) < lb
        }
        Real.from_rat(r) + Real.from_rat(eps) = Real.from_rat(r + eps)
        r < r + eps
        let b: Rat satisfy {
            r < b and Real.from_rat(b) < lb
        }
        false
    }
}

theorem cauchy_gt_is_dedekind_cut(q: Nat -> Real) {
    converges(q) implies is_dedekind_cut(cauchy_gt_rat(q))
} by {
    if converges(q) {
        is_lower(cauchy_gt_rat(q))
        not has_greatest(cauchy_gt_rat(q))
        is_dedekind_cut(cauchy_gt_rat(q))
    }
}

theorem limit_exists(q: Nat -> Real) {
    exists(lim: Real) {
        if converges(q) {
            Real.new(cauchy_gt_rat(q)) = Option.some(lim)
        } else {
            lim = Real.0
        }
    }
} by {
    if converges(q) {
        is_dedekind_cut(cauchy_gt_rat(q))
        Real.constraint(cauchy_gt_rat(q))
        let lim: Real satisfy {
            Real.new(cauchy_gt_rat(q)) = Option.some(lim)
        }
        exists(lim2: Real) {
            if converges(q) {
                Real.new(cauchy_gt_rat(q)) = Option.some(lim2)
            } else {
                lim2 = Real.0
            }
        }
    } else {
        exists(lim: Real) {
            if converges(q) {
                Real.new(cauchy_gt_rat(q)) = Option.some(lim)
            } else {
                lim = Real.0
            }
        }
    }
}

/// limit is defined with a "default" so that it's zero for things that don't converge.
let limit(q: Nat -> Real) -> lim: Real satisfy {
    if converges(q) {
        Real.new(cauchy_gt_rat(q)) = Option.some(lim)
    } else {
        lim = Real.0
    }
}

theorem eventual_lb_extends(q: Nat -> Real, lb1: Real, lb2: Real) {
    eventual_lb(q, lb1) and lb2 < lb1 implies eventual_lb(q, lb2)
} by {
    if eventual_lb(q, lb1) and lb2 < lb1 {
        lb2 <= lb1
        let n: Nat satisfy {
            forall(i: Nat) {
                n <= i implies lb1 <= q(i)
            }
        }
        forall(i: Nat) {
            if n <= i {
                lb1 <= q(i)
                lb2 <= q(i)
            }
        }
    }
}

theorem lt_limit_imp_lb(q: Nat -> Real, lb: Real) {
    converges(q) and lb < limit(q) implies eventual_lb(q, lb)
} by {
    if converges(q) and lb < limit(q) {
        let r: Rat satisfy {
            lb < Real.from_rat(r) and Real.from_rat(r) < limit(q)
        }
        lb < Real.from_rat(r)
        Real.from_rat(r) < limit(q)
        cauchy_gt_rat(q, r)
        exists(lb2: Real) {
            Real.from_rat(r) < lb2 and eventual_lb(q, lb2)
        }
        let lb2: Real satisfy {
            Real.from_rat(r) < lb2 and eventual_lb(q, lb2)
        }
        Real.from_rat(r) < lb2
        lb < lb2
        eventual_lb(q, lb2)
        eventual_lb(q, lb2) and lb < lb2 implies eventual_lb(q, lb)
        eventual_lb(q, lb)
    }
}

theorem lb_lte_limit(q: Nat -> Real, lb: Real) {
    converges(q) and eventual_lb(q, lb) implies lb <= limit(q)
} by {
    if converges(q) and eventual_lb(q, lb) {
        if not lb <= limit(q) {
            let r: Rat satisfy {
                lb.gt_rat(r) and not limit(q).gt_rat(r)
            }
            lb.gt_rat(r)
            lb > Real.from_rat(r)
            exists(lb2: Real) {
                lb2 > Real.from_rat(r) and eventual_lb(q, lb2)
            }
            cauchy_gt_rat(q, r)
            Option.some(limit(q)) = Real.new(cauchy_gt_rat(q))
            limit(q).gt_rat(r) = cauchy_gt_rat(q, r)
            limit(q).gt_rat(r)
            false
        }
    }
}

// The other direction of the lower bound.
// We can prove basically the same stuff here.
define eventual_ub(q: Nat -> Real, ub: Real) -> Bool {
    exists(n: Nat) {
        forall(i: Nat) {
            n <= i implies q(i) <= ub
        }
    }
}

theorem eventual_ub_extends(q: Nat -> Real, ub1: Real, ub2: Real) {
    eventual_ub(q, ub1) and ub1 < ub2 implies eventual_ub(q, ub2)
} by {
    if eventual_ub(q, ub1) and ub1 < ub2 {
        ub1 <= ub2
        let n: Nat satisfy {
            forall(i: Nat) {
                n <= i implies q(i) <= ub1
            }
        }
        forall(i: Nat) {
            if n <= i {
                q(i) <= ub1
                q(i) <= ub2
            }
        }
    }
}

define eventual_eq(q: Nat -> Real, a: Real) -> Bool {
    exists(n: Nat) {
        forall(i: Nat) {
            n <= i implies q(i) = a
        }
    }
}

theorem eventual_eq_imp_ub(q: Nat -> Real, a: Real) {
    eventual_eq(q, a) implies eventual_lb(q, a)
} by {
    let n: Nat satisfy {
        forall(i: Nat) {
            n <= i implies q(i) = a
        }
    }
    forall(i: Nat) {
        if n <= i {
            q(i) = a
            a <= q(i)
        }
    }
}

theorem eventual_eq_imp_lb(q: Nat -> Real, a: Real) {
    eventual_eq(q, a) implies eventual_ub(q, a)
} by {
    let n: Nat satisfy {
        forall(i: Nat) {
            n <= i implies q(i) = a
        }
    }
    forall(i: Nat) {
        if n <= i {
            q(i) = a
            q(i) <= a
        }
    }
}

theorem eq_converges(q: Nat -> Real, a: Real) {
    eventual_eq(q, a) implies converges(q)
} by {
    let n: Nat satisfy {
        forall(i: Nat) {
            n <= i implies q(i) = a
        }
    }
    forall(eps: Real) {
        if eps.is_positive {
            forall(i: Nat, j: Nat) {
                if n <= i and n <= j {
                    q(i) = a
                    q(j) = a
                    q(i).is_close(q(j), eps)
                }
            }
            cauchy_bound(q, n, eps)
        }
    }
}

theorem lb_ub_imp_eq(q: Nat -> Real, a: Real) {
    eventual_lb(q, a) and eventual_ub(q, a) implies eventual_eq(q, a)
} by {
    let n1: Nat satisfy {
        forall(i: Nat) {
            n1 <= i implies q(i) <= a
        }
    }
    let n2: Nat satisfy {
        forall(i: Nat) {
            n2 <= i implies q(i) >= a
        }
    }
    let n: Nat satisfy {
        n1 <= n and n2 <= n
    }
    forall(i: Nat) {
        if n <= i {
            q(i) >= a
            q(i) = a
        }
    }
}

theorem lb_lte_ub(q: Nat -> Real, lb: Real, ub: Real) {
    eventual_lb(q, lb) and eventual_ub(q, ub)
    implies
    lb <= ub
} by {
    let n1: Nat satisfy {
        forall(i: Nat) {
            n1 <= i implies lb <= q(i)
        }
    }
    let n2: Nat satisfy {
        forall(i: Nat) {
            n2 <= i implies q(i) <= ub
        }
    }
    let k: Nat satisfy {
        n1 <= k and n2 <= k
    }
    lb <= q(k)
    q(k) <= ub
}

theorem ub_imp_limit_lte(q: Nat -> Real, ub: Real) {
    converges(q) and eventual_ub(q, ub) implies limit(q) <= ub
} by {
    if converges(q) and eventual_ub(q, ub) {
        if limit(q) > ub {
            let r: Rat satisfy {
                limit(q) > Real.from_rat(r) and Real.from_rat(r) > ub
            }
            Real.from_rat(r) < limit(q)
            converges(q) and Real.from_rat(r) < limit(q)
            eventual_lb(q, Real.from_rat(r))
            eventual_lb(q, Real.from_rat(r)) and eventual_ub(q, ub)
            Real.from_rat(r) <= ub
            not Real.from_rat(r) <= ub
            false
        }
    }
}

// Relate the cauchy condition to eventual bounds

theorem cauchy_imp_lb(q: Nat -> Real, n: Nat, eps: Real) {
    cauchy_bound(q, n, eps) implies eventual_lb(q, q(n) - eps)
} by {
    forall(i: Nat) {
        if n <= i {
            q(n) - eps <= q(i)
        }
    }
}

theorem cauch_imp_ub(q: Nat -> Real, n: Nat, eps: Real) {
    cauchy_bound(q, n, eps) implies eventual_ub(q, q(n) + eps)
} by {
    forall(i: Nat) {
        if n <= i {
            q(i).is_close(q(n), eps)
            q(i) <= q(n) + eps
        }
    }
}

theorem lt_imp_minus_pos(a: Real, b: Real) {
    a < b implies (b - a).is_positive
} by {
    if a < b {
        a + -b < b + -b
        b + -b = Real.0
        a + -b < Real.0
        (a + -b).is_negative
    }
}

theorem from_rat_pos(r: Rat) {
    r.is_positive implies Real.from_rat(r).is_positive
}

theorem sub_lte(a: Real, b: Real, c: Real) {
    a - c <= b implies a <= b + c
} by {
    a - c + c <= b + c
}

theorem limit_lt_imp_ub(q: Nat -> Real, ub: Real) {
    converges(q) and limit(q) < ub implies eventual_ub(q, ub)
} by {
    Real.0 < ub - limit(q)
    exists(eps2: Rat) {
        eps2.is_positive and Real.0 + Real.from_rat(eps2) < (ub - limit(q))
    }
    let eps2: Rat satisfy {
        eps2.is_positive and Real.from_rat(eps2) < (ub - limit(q))
    }
    let eps = Real.from_rat(eps2 / Rat.2)
    eps + eps + limit(q) < ub - limit(q) + limit(q)
    limit(q) + eps + eps < ub

    let n: Nat satisfy {
        cauchy_bound(q, n, eps)
    }
    if q(n) + eps = limit(q) + eps + eps {
    } else {
        q(n) + eps < limit(q) + eps + eps
        eventual_ub(q, ub)
    }
}

/// The limit of a convergent sequence in a union of two separated closed rays
/// stays in that union.
theorem limit_in_union(seq: Nat -> Real, a: Real, b: Real) {
    converges(seq) and a < b and (forall(k: Nat) { seq(k) <= a or b <= seq(k) })
    implies
    limit(seq) <= a or b <= limit(seq)
} by {
    if converges(seq) and a < b and (forall(k: Nat) { seq(k) <= a or b <= seq(k) }) {
        if not (limit(seq) <= a) {
            a < limit(seq)
            let r: Rat satisfy { a < Real.from_rat(r) and Real.from_rat(r) < limit(seq) }
            eventual_lb(seq, Real.from_rat(r))
            let n: Nat satisfy { forall(i: Nat) { n <= i implies Real.from_rat(r) <= seq(i) } }
            forall(i: Nat) {
                if n <= i {
                    Real.from_rat(r) <= seq(i)
                    not (seq(i) <= a)
                    b <= seq(i)
                }
            }
            eventual_lb(seq, b)
            b <= limit(seq)
        }
    }
}

theorem eq_imp_limit(q: Nat -> Real, a: Real) {
    eventual_eq(q, a) implies limit(q) = a
}

// The bound on the end of a sequence that's part of the Weierstress convergence definition.
// Separating it out like this makes convergence a bit clearer to prove.
define tail_bound(q: Nat -> Real, a: Real, n: Nat, eps: Real) -> Bool {
    forall(i: Nat) {
        n <= i implies q(i).is_close(a, eps)
    }
}

theorem tail_bound_implies_is_close(q: Nat -> Real, a: Real, n: Nat, eps: Real, i: Nat) {
    tail_bound(q, a, n, eps) and n <= i implies q(i).is_close(a, eps)
} by {
    tail_bound(q, a, n, eps) = forall(j: Nat) {
        n <= j implies q(j).is_close(a, eps)
    }
    if n <= i {
        forall(j: Nat) {
            n <= j implies q(j).is_close(a, eps)
        }
        n <= i implies q(i).is_close(a, eps)
        q(i).is_close(a, eps)
    }
}

// converges_to is the Weierstrass definition.
define converges_to(q: Nat -> Real, a: Real) -> Bool {
    forall(eps: Real) {
        eps.is_positive implies exists(n: Nat) {
            tail_bound(q, a, n, eps)
        }
    }
}

theorem lt_converges_to_imp_lb(q: Nat -> Real, a: Real, b: Real) {
    converges_to(q, b) and a < b implies eventual_lb(q, a)
} by {
    if converges_to(q, b) and a < b {
        let eps = b - a
        a + eps = b
        a = b - eps
        eps.is_positive
        exists(n: Nat) {
            tail_bound(q, b, n, eps)
        }
        let n: Nat satisfy {
            tail_bound(q, b, n, eps)
        }
        forall(i: Nat) {
            if n <= i {
                q(i).is_close(b, eps)
                q(i) > b - eps
                q(i) > a
                q(i) >= a
                a <= q(i)
            }
        }
        eventual_lb(q, a)
    }
}

theorem gt_converges_to_imp_ub(q: Nat -> Real, a: Real, b: Real) {
    converges_to(q, b) and a > b implies eventual_ub(q, a)
} by {
    if converges_to(q, b) and a > b {
        let eps = a - b
        a = b + eps
        eps.is_positive
        exists(n: Nat) {
            tail_bound(q, b, n, eps)
        }
        let n: Nat satisfy {
            tail_bound(q, b, n, eps)
        }
        forall(i: Nat) {
            if n <= i {
                q(i).is_close(b, eps)
                q(i) < b + eps
                q(i) < a
                q(i) <= a
            }
        }
    }
}

theorem eps_lt_half(a: Real) {
    a.is_positive implies exists(b: Real) {
        b.is_positive and b + b < a
    }
} by {
    let c: Rat satisfy {
        Real.0 < Real.from_rat(c) and Real.from_rat(c) < a
    }
    let b = c / Rat.2
    Real.from_rat(c) <= Real.from_rat(c).abs
    Real.from_rat(c).abs = Real.from_rat(c.abs)
    Real.0 < Real.from_rat(c.abs)
    c.is_positive
    Real.from_rat(b) + Real.from_rat(b) = Real.from_rat(c)
}

theorem add_gt_imp_gt_sub(a: Real, b: Real, c: Real) {
    a + b > c implies a > c - b
} by {
    a + b + -b > c + -b
}

theorem sub_either_order(a: Real, b: Real, c: Real) {
    a - b - c = a - c - b
}

theorem sub_both_eq_sub_add(a: Real, b: Real, c: Real) {
    a - b - c = a - (b + c)
}

theorem is_close_triangle(a: Real, b: Real, c: Real, ace: Real, bce: Real) {
    a.is_close(c, ace) and b.is_close(c, bce)
    implies
    a.is_close(b, ace + bce)
} by {
    // First let's lower-bound a
    a + ace > b - bce

    // Then let's upper-bound a
    c + ace < b + bce + ace
    a < b + ace + bce

}

// Prove our two definitions of convergence are equivalent.

theorem converges_to_imp_converges(q: Nat -> Real, a: Real) {
    converges_to(q, a) implies converges(q)
} by {
    if converges_to(q, a) {
        forall(eps: Real) {
            if eps.is_positive {
                let eps2: Real satisfy {
                    eps2.is_positive and eps2 + eps2 < eps
                }
                eps2.is_positive
                exists(n: Nat) {
                    tail_bound(q, a, n, eps2)
                }
                let n: Nat satisfy {
                    tail_bound(q, a, n, eps2)
                }
                forall(i: Nat, j: Nat) {
                    if n <= i and n <= j {
                        q(i).is_close(a, eps2)
                        q(j).is_close(a, eps2)
                        q(i).is_close(q(j), eps2 + eps2)
                        q(i).is_close(q(j), eps)
                    }
                }
                exists(k0: Nat, k1: Nat) {
                    n <= k0 and n <= k1 and not q(k0).is_close(q(k1), eps)
                } or cauchy_bound(q, n, eps)
                cauchy_bound(q, n, eps)
            }
        }
    }
}

theorem sub_lt_is_gt(a: Real, b: Real, c: Real) {
    b < c implies a - b > a - c
} by {
    if b < c {
        (c - b).is_positive
        a < a + (c - b)
        c - b + a = a + (c - b)
        a + c - b = a - b + c
        a - b + c > a
    }
}

theorem smaller_pos(a: Real) {
    a.is_positive implies exists(b: Real) {
        b.is_positive and b < a
    }
} by {
    if a.is_positive {
        a > Real.0
        exists(r: Rat) {
            a > Real.from_rat(r) and Real.from_rat(r) > Real.0
        }
        let r: Rat satisfy {
            a > Real.from_rat(r) and Real.from_rat(r) > Real.0
        }
        Real.from_rat(r) > Real.0
        Real.from_rat(r).is_positive
        exists(b: Real) {
            b.is_positive and b < a
        }
    }
}

theorem converges_imp_converges_to(q: Nat -> Real) {
    converges(q) implies converges_to(q, limit(q))
} by {
    forall(eps: Real) {
        if eps.is_positive {
            let eps2: Real satisfy {
                eps2.is_positive and eps2 < eps
            }

            // Find a lower bound
            let lb = limit(q) - eps2
            lb > limit(q) - eps
            limit(q).is_close(limit(q), eps2)
            lb < limit(q)
            eventual_lb(q, lb)
            let n1: Nat satisfy {
                forall(i: Nat) {
                    n1 <= i implies q(i) >= lb
                }
            }

            // Find an upper bound
            let ub = limit(q) + eps2
            let n2: Nat satisfy {
                forall(i: Nat) {
                    n2 <= i implies q(i) <= ub
                }
            }

            // Once both of those are hit, the Weierstrass condition is ok
            let n: Nat satisfy {
                n1 <= n and n2 <= n
            }
            forall(i: Nat) {
                if n <= i {
                    q(i) >= lb
                    q(i) > limit(q) - eps
                    q(i) <= ub
                    q(i) < limit(q) + eps
                    q(i).is_close(limit(q), eps)
                }
            }
            tail_bound(q, limit(q), n, eps)
        }
    }
}

theorem converges_to_unique_one_way(q: Nat -> Real, a: Real, b: Real) {
    converges_to(q, a) and converges_to(q, b) implies not a < b
} by {
    if a < b {
        let r1: Real satisfy {
            a < r1 and r1 < b
        }
        let r2q: Rat satisfy {
            r1 < Real.from_rat(r2q) and Real.from_rat(r2q) < b
        }
        let r2 = Real.from_rat(r2q)
        r2 <= r1
        false
    }
}

theorem converges_to_unique(q: Nat -> Real, a: Real, b: Real) {
    converges_to(q, a) and converges_to(q, b) implies a = b
} by {
    if b < a {
        false
    }
}

/// Alias endpoint: limits of convergent real sequences are unique.
theorem limit_unique(q: Nat -> Real, a: Real, b: Real) {
    converges_to(q, a) and converges_to(q, b) implies a = b
} by {
    converges_to_unique(q, a, b)
}

/// Alias endpoint: convergence to a specified value implies Cauchy convergence.
theorem converges_to_is_convergent(q: Nat -> Real, a: Real) {
    converges_to(q, a) implies converges(q)
} by {
    converges_to_imp_converges(q, a)
}

/// Alias endpoint: an eventually constant sequence converges.
theorem eventual_eq_converges(q: Nat -> Real, a: Real) {
    eventual_eq(q, a) implies converges(q)
} by {
    eq_converges(q, a)
}

/// Alias endpoint: an eventually constant sequence has the eventual value as limit.
theorem eventual_eq_limit(q: Nat -> Real, a: Real) {
    eventual_eq(q, a) implies limit(q) = a
} by {
    eq_imp_limit(q, a)
}

/// Alias endpoint: every convergent sequence converges to its canonical limit.
theorem convergent_converges_to_limit(q: Nat -> Real) {
    converges(q) implies converges_to(q, limit(q))
} by {
    converges_imp_converges_to(q)
}

// Lifting a sequence of rationals to a sequence of reals
define lift_seq(q: Nat -> Rat) -> Nat -> Real {
    compose(Real.from_rat, q)
}

theorem lift_seq_elt(q: Nat -> Rat, n: Nat) {
    lift_seq(q)(n) = Real.from_rat(q(n))
} by {
    lift_seq(q)(n) = compose(Real.from_rat, q)(n)
}

let rat_approx(x: Real, eps: Rat) -> r: Rat satisfy {
    eps.is_positive implies
    x.is_close(Real.from_rat(r), Real.from_rat(eps))
}

// A sequence of rationals that approximates a real number.
// Chosen to be easy to calculate with.
define rat_seq(x: Real, n: Nat) -> Rat {
    rat_approx(x, iop(n))
}

theorem rat_seq_is_close(x: Real, n: Nat) {
    x.is_close(Real.from_rat(rat_seq(x, n)), Real.from_rat(iop(n)))
}

theorem abs_lt_imp_close_to_zero(x: Real, eps: Real) {
    x.abs < eps implies x.is_close(Real.0, eps)
} by {
    if x.is_negative {
    } else {
    }
}

theorem iop_limit {
    converges_to(lift_seq(iop), Real.0)
} by {
    forall(eps: Real) {
        if eps.is_positive {
            Real.0 < eps
            exists(reps: Rat) {
                reps.is_positive and Real.0 + Real.from_rat(reps) < eps
            }
            let reps: Rat satisfy {
                reps.is_positive and Real.from_rat(reps) < eps
            }
            let n: Nat satisfy {
                forall(i: Nat) {
                    n <= i implies iop(i) < reps
                }
            }

            forall(i: Nat) {
                if n <= i {
                    Real.from_rat(iop(i)).abs = Real.from_rat(iop(i))
                    Real.from_rat(iop(i)) = lift_seq(iop, i)
                    Real.from_rat(iop(i)) < Real.from_rat(reps)
                    Real.from_rat(iop(i)) < eps
                    lift_seq(iop, i).abs < eps
                    lift_seq(iop)(i).is_close(Real.0, eps)
                }
            }
            tail_bound(lift_seq(iop), Real.0, n, eps)
        }
    }
}

define add_seq(a: Nat -> Real, b: Nat -> Real, n: Nat) -> Real {
    a(n) + b(n)
}

theorem half_pos_is_pos(a: Real, b: Real) {
    b.is_positive and a + a = b
    implies
    a.is_positive
}

theorem add_close(a1: Real, b1: Real, a2: Real, b2: Real, a_eps: Real, b_eps: Real) {
    a1.is_close(a2, a_eps) and b1.is_close(b2, b_eps)
    implies
    (a1 + b1).is_close(a2 + b2, a_eps + b_eps)
} by {
    // The less-than direction
    a1 + b1 < a2 + a_eps + b2 + b_eps

    // The greater-than direction
    a2 - a_eps < a1
    b2 - b_eps < b1
    a2 - a_eps + (b2 - b_eps) < a1 + b1
    a2 + b2 - (a_eps + b_eps) = a2 + b2 - a_eps - b_eps
    a2 + b2 - a_eps = a2 - a_eps + b2
    a1 + b1 > (a2 + b2) - (a_eps + b_eps)
}

theorem limit_add_seq(a: Nat -> Real, b: Nat -> Real) {
    converges(a) and converges(b)
    implies
    converges_to(add_seq(a, b), limit(a) + limit(b))
} by {
    let q = add_seq(a, b)
    let ql = limit(a) + limit(b)
    forall(eps: Real) {
        if eps.is_positive {
            let eps2: Real satisfy {
                eps2.is_positive and eps2 + eps2 < eps
            }

            converges_to(a, limit(a))
            exists(n1: Nat) {
                tail_bound(a, limit(a), n1, eps2)
            }
            let n1: Nat satisfy {
                tail_bound(a, limit(a), n1, eps2)
            }

            converges_to(b, limit(b))
            exists(n2: Nat) {
                tail_bound(b, limit(b), n2, eps2)
            }
            let n2: Nat satisfy {
                tail_bound(b, limit(b), n2, eps2)
            }

            let n: Nat satisfy {
                n1 <= n and n2 <= n
            }

            forall(i: Nat) {
                if n <= i {
                    a(i).is_close(limit(a), eps2)
                    b(i).is_close(limit(b), eps2)
                    (a(i) + b(i)).is_close(limit(a) + limit(b), eps2 + eps2)
                    q(i).is_close(ql, eps)
                }
            }
            tail_bound(q, ql, n, eps)
        }
    }
    converges_to(q, ql)
}

theorem lift_rat_seq_close(x: Real, n: Nat) {
    lift_seq(rat_seq(x))(n).is_close(x, Real.from_rat(iop(n)))
}

theorem rat_seq_converges_to(x: Real) {
    converges_to(lift_seq(rat_seq(x)), x)
} by {
    forall(eps: Real) {
        if eps.is_positive {
            Real.0 < eps
            exists(reps: Rat) {
                reps.is_positive and Real.0 + Real.from_rat(reps) < eps
            }
            let reps: Rat satisfy {
                Real.from_rat(reps) < eps and reps.is_positive
            }
            let n: Nat satisfy {
                forall(i: Nat) {
                    n <= i implies iop(i) < reps
                }
            }

            forall(i: Nat) {
                if n <= i {
                    Real.from_rat(iop(i)) < eps
                    (Real.from_rat(rat_seq(x, i)) - x).abs < eps
                    lift_seq(rat_seq(x))(i) = Real.from_rat(rat_seq(x, i))
                    lift_seq(rat_seq(x))(i).is_close(x, eps)
                }
            }
            tail_bound(lift_seq(rat_seq(x)), x, n, eps)
        }
    }
}

theorem rat_close_to_both_lt(x: Real, y: Real, eps: Real) {
    x.is_close(y, eps) and x < y implies
    exists(r: Rat) {
        x.is_close(Real.from_rat(r), eps)
        and
        y.is_close(Real.from_rat(r), eps)
    }
} by {
    let r: Rat satisfy {
        x < Real.from_rat(r) and Real.from_rat(r) < y
    }
    y < Real.from_rat(r) + eps
    x.is_close(Real.from_rat(r), eps)
}

theorem rat_close(x: Real, y: Real, eps: Real) {
    x.is_close(y, eps) implies
    exists(r: Rat) {
        x.is_close(Real.from_rat(r), eps)
        and
        y.is_close(Real.from_rat(r), eps)
    }
} by {
    if x < y {
        exists(r: Rat) {
            x.is_close(Real.from_rat(r), eps)
            and
            y.is_close(Real.from_rat(r), eps)
        }
    } else {
        if y < x {
            y.is_close(x, eps)
            let r: Rat satisfy {
                y.is_close(Real.from_rat(r), eps)
                and
                x.is_close(Real.from_rat(r), eps)
            }
            exists(r2: Rat) {
                x.is_close(Real.from_rat(r2), eps)
                and
                y.is_close(Real.from_rat(r2), eps)
            }
        } else {
            x = y
            x.is_close(x, eps)
            exists(r1: Rat) {
                Real.from_rat(r1).is_close(x, eps) and Real.from_rat(r1).is_close(y, eps)
            }
            let r1: Rat satisfy {
                Real.from_rat(r1).is_close(x, eps) and Real.from_rat(r1).is_close(y, eps)
            }
            x.is_close(Real.from_rat(r1), eps)
            y.is_close(Real.from_rat(r1), eps)
            exists(r2: Rat) {
                x.is_close(Real.from_rat(r2), eps)
                and
                y.is_close(Real.from_rat(r2), eps)
            }
        }
    }
}

theorem lift_rat_seq_converges(x: Real) {
    converges(lift_seq(rat_seq(x)))
}

// When two sequences get close to each other eventually.
define seq_close(a: Nat -> Real, b: Nat -> Real, eps: Real) -> Bool {
    exists(n: Nat) {
        forall(i: Nat) {
            n <= i implies a(i).is_close(b(i), eps)
        }
    }
}

theorem convergent_seqs_get_close(a: Nat -> Real, b: Nat -> Real, eps: Real) {
    eps.is_positive and converges(a) and converges(b) and limit(a) = limit(b)
    implies
    seq_close(a, b, eps)
} by {
    let eps2: Real satisfy {
        eps2.is_positive and eps2 + eps2 < eps
    }
    converges_to(a, limit(a))
    exists(n1: Nat) {
        tail_bound(a, limit(a), n1, eps2)
    }
    let n1: Nat satisfy {
        tail_bound(a, limit(a), n1, eps2)
    }
    converges_to(b, limit(b))
    exists(n2: Nat) {
        tail_bound(b, limit(b), n2, eps2)
    }
    let n2: Nat satisfy {
        tail_bound(b, limit(b), n2, eps2)
    }
    let n: Nat satisfy {
        n1 <= n and n2 <= n
    }
    forall(i: Nat) {
        if n <= i {
            b(i).is_close(limit(b), eps2)
            a(i).is_close(b(i), eps2 + eps2)
            a(i).is_close(b(i), eps)
        }
    }
    forall(i: Nat) {
        n <= i implies a(i).is_close(b(i), eps)
    }
    exists(n0: Nat) {
        forall(i: Nat) {
            n0 <= i implies a(i).is_close(b(i), eps)
        }
    }
    seq_close(a, b, eps)
}

theorem only_abs_zero_eq_zero(x: Real) {
    x.abs = Real.0 implies x = Real.0
} by {
    if x.abs = Real.0 {
        if x.is_negative {
            -x = x.abs
            --x = x
        } else {
            x.abs = x
        }
    }
}

theorem close_limit_imp_seqs_get_close(a: Nat -> Real, b: Nat -> Real, eps: Real) {
    converges(a) and converges(b) and limit(a).is_close(limit(b), eps)
    implies
    seq_close(a, b, eps)
} by {
    if limit(a) = limit(b) {
    } else {
        let diff = (limit(a) - limit(b)).abs
        diff < eps
        let eps2: Real satisfy {
            diff < eps2 and eps2 < eps
        }
        exists(reps: Rat) {
            reps.is_positive and eps2 + Real.from_rat(reps) < eps
        }
        let reps: Rat satisfy {
            reps.is_positive and eps2 + Real.from_rat(reps) < eps
        }
        exists(eps3: Real) {
            eps3.is_positive and eps2 + eps3 < eps
        }
        let eps3: Real satisfy {
            eps3.is_positive and eps2 + eps3 < eps
        }
        exists(eps4: Real) {
            eps4.is_positive and eps4 + eps4 < eps3
        }
        let eps4: Real satisfy {
            eps4.is_positive and eps4 + eps4 < eps3
        }
        converges_to(a, limit(a))
        exists(n1: Nat) {
            tail_bound(a, limit(a), n1, eps4)
        }
        let n1: Nat satisfy {
            tail_bound(a, limit(a), n1, eps4)
        }
        converges_to(b, limit(b))
        exists(n2: Nat) {
            tail_bound(b, limit(b), n2, eps4)
        }
        let n2: Nat satisfy {
            tail_bound(b, limit(b), n2, eps4)
        }
        let n: Nat satisfy {
            n1 <= n and n2 <= n
        }
        forall(i: Nat) {
            if n <= i {
                a(i).is_close(limit(a), eps4)
                limit(a).is_close(limit(b), eps2)
                a(i).is_close(limit(b), eps4 + eps2)
                b(i).is_close(limit(b), eps4)
                a(i).is_close(b(i), eps4 + eps2 + eps4)
                a(i).is_close(b(i), eps4 + eps4 + eps2)
                eps4 + eps4 + eps2 < eps3 + eps2
                eps3 + eps2 = eps2 + eps3
                eps4 + eps4 + eps2 < eps
                (a(i) - b(i)).abs < eps
                a(i).is_close(b(i), eps)
            }
        }
    }
}

theorem close_and_lt_imp_close(x: Real, y: Real, eps1: Real, eps2: Real) {
    x.is_close(y, eps1) and eps1 < eps2 implies x.is_close(y, eps2)
}

theorem find_less_than_a_third(x: Real) {
    x.is_positive implies
    exists(eps: Real) {
        eps.is_positive and eps + eps + eps < x
    }
} by {
    Real.0 < x
    exists(r: Rat) {
        r.is_positive and Real.0 + Real.from_rat(r) < x
    }
    let r: Rat satisfy {
        r.is_positive and Real.from_rat(r) < x
    }
    let reps = r / Rat.3
    Real.from_rat(reps) + Real.from_rat(reps) + Real.from_rat(reps) < x
}

define limit_rat(a: Nat -> Rat) -> Real {
    limit(lift_seq(a))
}

theorem neq_imp_abs_diff_pos(x: Real, y: Real) {
    x != y implies (x - y).abs.is_positive
} by {
    x - y = x + -y
    --y = y
    Real.0 + --y = --y
    Real.0 - -y = Real.0 + --y
    x + -y != Real.0 or Real.0 - -y = x
    (x + -y).abs != Real.0 or x + -y = Real.0
    (x - y).abs.is_negative or (x - y).abs.is_positive or (x - y).abs = Real.0
    if x != y {
        (x - y).abs != Real.0
        not (x - y).abs.is_negative
    }
}

theorem eps_smaller_than_both(eps1: Real, eps2: Real) {
    eps1.is_positive and eps2.is_positive implies
    exists(eps3: Real) {
        eps3.is_positive and eps3 < eps1 and eps3 < eps2
    }
} by {
    Real.0 < eps1.min(eps2)
    let eps3: Real satisfy {
        Real.0 < eps3 and eps3 < eps1.min(eps2)
    }
}

// Equivalence classes between Cauchy sequences.
define eq_seq(a: Nat -> Rat, b: Nat -> Rat) -> Bool {
    converges(lift_seq(a)) and converges_to(lift_seq(b), limit_rat(a))
}

theorem eq_seq_symm(a: Nat -> Rat, b: Nat -> Rat) {
    eq_seq(a, b) implies eq_seq(b, a)
} by {
    if eq_seq(a, b) {
        converges(lift_seq(a))
        converges_to(lift_seq(b), limit_rat(a))
        converges(lift_seq(b))
        converges_to(lift_seq(b), limit(lift_seq(b)))
        limit(lift_seq(b)) = limit_rat(b)
        limit(lift_seq(b)) = limit_rat(a)
        converges_to(lift_seq(a), limit(lift_seq(a)))
        limit(lift_seq(a)) = limit_rat(a)
        converges_to(lift_seq(a), limit_rat(b))
    }
}

define add_rat_seq(a: Nat -> Rat, b: Nat -> Rat, n: Nat) -> Rat {
    a(n) + b(n)
}

theorem add_rat_seq_is_add(a: Nat -> Rat, b: Nat -> Rat) {
    converges(lift_seq(a)) and converges(lift_seq(b))
    implies
    converges_to(lift_seq(add_rat_seq(a, b)), limit_rat(a) + limit_rat(b))
} by {
    let seq = lift_seq(add_rat_seq(a, b))
    let target = limit_rat(a) + limit_rat(b)
    forall(eps: Real) {
        if eps.is_positive {
            let delta: Real satisfy {
                delta.is_positive and delta + delta < eps
            }
            let n1: Nat satisfy {
                tail_bound(lift_seq(a), limit_rat(a), n1, delta)
            }
            let n2: Nat satisfy {
                tail_bound(lift_seq(b), limit_rat(b), n2, delta)
            }
            let n: Nat satisfy {
                n1 <= n and n2 <= n
            }
            forall(i: Nat) {
                if n <= i {
                    lift_seq(b)(i).is_close(limit_rat(b), delta)
                    (lift_seq(a, i) + lift_seq(b, i)).is_close(limit_rat(a) + limit_rat(b), delta + delta)
                    Real.from_rat(a(i)) = lift_seq(a, i)
                    Real.from_rat(b(i)) = lift_seq(b, i)
                    Real.from_rat(add_rat_seq(a, b, i)) = lift_seq(add_rat_seq(a, b), i)
                    seq(i).is_close(target, delta + delta)
                    seq(i).is_close(target, eps)
                }
            }
            tail_bound(seq, target, n, eps)
        }
    }
    converges_to(seq, target)
}

define mul_rat_seq(a: Nat -> Rat, b: Nat -> Rat, n: Nat) -> Rat {
    a(n) * b(n)
}

define neg_rat_seq(a: Nat -> Rat, n: Nat) -> Rat {
    -a(n)
}

theorem neg_is_close(x: Real, y: Real, eps: Real) {
    x.is_close(y, eps) implies
    (-x).is_close(-y, eps)
} by {
    (-x + y).abs < eps
}

theorem neg_seq_converges(a: Nat -> Rat) {
    converges(lift_seq(a)) implies
    converges_to(lift_seq(neg_rat_seq(a)), -limit_rat(a))
} by {
    let seq = lift_seq(neg_rat_seq(a))
    let target = -limit_rat(a)
    forall(eps: Real) {
        if eps.is_positive {
            let delta: Real satisfy {
                delta.is_positive and delta < eps
            }
            let n1: Nat satisfy {
                tail_bound(lift_seq(a), limit_rat(a), n1, delta)
            }
            let n: Nat satisfy {
                n1 <= n
            }
            forall(i: Nat) {
                if n <= i {
                    (-lift_seq(a, i)).is_close(-limit_rat(a), delta)
                    seq(i).is_close(target, eps)
                }
            }
            tail_bound(seq, target, n, eps)
        }
    }
    converges_to(seq, target)
}

// Prove that convergent sequences can be bounded.
theorem converges_imp_bounded(a: Nat -> Rat) {
    converges(lift_seq(a))
    implies
    exists(b: Rat) {
        forall(i: Nat) {
            a(i) < b
        }
    }
} by {
    let eventual: Real satisfy {
        eventual_ub(lift_seq(a), eventual)
    }
    let n: Nat satisfy {
        forall(i: Nat) {
            n <= i implies lift_seq(a)(i) <= eventual
        }
    }
    exists(abs_bound: Rat) {
        forall(i: Nat) {
            i <= n implies a(i).abs < abs_bound
        }
    }
    exists(finite: Rat) {
        forall(i: Nat) {
            i <= n implies a(i) < finite
        }
    }
    let finite: Rat satisfy {
        forall(i: Nat) {
            i <= n implies a(i) < finite
        }
    }
    let eventual_bound: Rat satisfy {
        eventual < Real.from_rat(eventual_bound)
    }
    let finite_bound: Rat satisfy {
        finite < finite_bound
    }
    if eventual_bound <= finite_bound {
        Real.from_rat(eventual_bound) <= Real.from_rat(finite_bound)
        eventual < Real.from_rat(finite_bound)
        exists(bound: Rat) {
            eventual < Real.from_rat(bound) and finite < bound
        }
    } else {
        finite < eventual_bound
        exists(bound: Rat) {
            eventual < Real.from_rat(bound) and finite < bound
        }
    }
    let bound: Rat satisfy {
        eventual < Real.from_rat(bound) and finite < bound
    }
    forall(i: Nat) {
        if n <= i {
            Real.from_rat(bound).gt_rat(a(i))
            a(i) < bound
        } else {
            i <= n
            a(i) < finite
            finite < bound
            a(i) < bound
        }
    }
}

// Prove that convergent sequences can have their absolute value bounded.
theorem converges_imp_abs_bounded(a: Nat -> Rat) {
    converges(lift_seq(a))
    implies
    exists(b: Rat) {
        forall(i: Nat) {
            a(i).abs < b
        }
    }
} by {
    let pos_bound: Rat satisfy {
        forall(i: Nat) {
            a(i) < pos_bound
        }
    }
    converges(lift_seq(neg_rat_seq(a)))
    let neg_bound: Rat satisfy {
        forall(i: Nat) {
            neg_rat_seq(a)(i) < neg_bound
        }
    }
    let bound: Rat satisfy {
        pos_bound < bound and neg_bound < bound
    }
    forall(i: Nat) {
        if a(i).is_negative {
            a(i).abs = -a(i)
            neg_rat_seq(a, i) = -a(i)
            a(i).abs < bound
        } else {
            a(i).abs < bound
        }
    }
}

theorem diff_pos_imp_lt(a: Real, b: Real) {
    (b - a).is_positive implies a < b
} by {
    if (b - a).is_positive {
        b + -a = b - a
        b + -a = -a + b
        -a + b > Real.0
        -a > Real.0 - b
        Real.0 - b = Real.0 + -b
        Real.0 + -b = -b
        -b < -a
    }
}

theorem smaller_rat_eps(eps: Real) {
    eps.is_positive implies
    exists(reps: Rat) {
        reps.is_positive and Real.from_rat(reps) < eps
    }
} by {
    Real.0 < eps
    exists(reps: Rat) {
        reps.is_positive and Real.0 + Real.from_rat(reps) < eps
    }
}

// Prove that multiplying convergent sequences gives you a convergent sequence.
theorem mul_rat_seq_converges(a: Nat -> Rat, b: Nat -> Rat) {
    converges(lift_seq(a)) and converges(lift_seq(b))
    implies
    converges(lift_seq(mul_rat_seq(a, b)))
} by {
    let a_ub: Rat satisfy {
        forall(i: Nat) {
            a(i).abs < a_ub
        }
    }
    let b_ub: Rat satisfy {
        forall(i: Nat) {
            b(i).abs < b_ub
        }
    }
    let ub: Rat satisfy {
        a_ub < ub and b_ub < ub
    }
    let seq = lift_seq(mul_rat_seq(a, b))
    forall(eps: Real) {
        if eps.is_positive {
            let reps: Rat satisfy {
                reps.is_positive and Real.from_rat(reps) < eps
            }
            let reps2 = reps / (ub + ub)
            (ub + ub).is_positive
            reps2.is_positive
            let eps2 = Real.from_rat(reps2)
            eps2.is_positive
            exists(n_a: Nat) {
                cauchy_bound(lift_seq(a), n_a, eps2)
            }
            let n_a: Nat satisfy {
                cauchy_bound(lift_seq(a), n_a, eps2)
            }
            let lsb = lift_seq(b)
            exists(n_b: Nat) {
                cauchy_bound(lsb, n_b, eps2)
            }
            let n_b: Nat satisfy {
                cauchy_bound(lsb, n_b, eps2)
            }
            let n: Nat satisfy {
                n_a <= n and n_b <= n
            }
            forall(i: Nat, j: Nat) {
                if n <= i and n <= j {
                    lift_seq(a)(i).is_close(lift_seq(a)(j), eps2)
                    lsb(i).is_close(lsb(j), eps2)
                    (b(i) - b(j)).abs < reps2
                    (a(i) - a(j)).abs < reps2
                    b(j).abs * (a(i) - a(j)).abs < ub * reps2
                    a(i).abs * (b(i) - b(j)).abs + b(j).abs * (a(i) - a(j)).abs < ub * reps2 + ub * reps2
                    let left = (a(i) * b(i) - a(j) * b(j)).abs
                    let middle = a(i).abs * (b(i) - b(j)).abs + b(j).abs * (a(i) - a(j)).abs
                    let right = ub * reps2 + ub * reps2
                    left <= middle
                    (ub + ub) * (reps / (ub + ub)) = reps
                    (a(i) * b(i) - a(j) * b(j)).abs < reps
                    (a(i) * b(i)).is_close(a(j) * b(j), reps)
                    lift_seq(mul_rat_seq(a, b))(i).is_close(lift_seq(mul_rat_seq(a, b))(j), Real.from_rat(reps))
                    seq(i).is_close(seq(j), eps)
                }
            }
            exists(k0: Nat, k1: Nat) {
                n <= k0 and n <= k1 and not seq(k0).is_close(seq(k1), eps)
            } or cauchy_bound(seq, n, eps)
            cauchy_bound(seq, n, eps)
        }
    }
}

define sub_rat_seq(a: Nat -> Rat, b: Nat -> Rat, n: Nat) -> Rat {
    a(n) - b(n)
}

theorem add_neg_is_sub(a: Nat -> Rat, b: Nat -> Rat) {
    add_rat_seq(a, neg_rat_seq(b)) = sub_rat_seq(a, b)
} by {
    forall(n: Nat) {
        a(n) + neg_rat_seq(b, n) = add_rat_seq(a, neg_rat_seq(b), n)
        -b(n) = neg_rat_seq(b, n)
        a(n) + -b(n) = a(n) - b(n)
        a(n) - b(n) = sub_rat_seq(a, b, n)
        add_rat_seq(a, neg_rat_seq(b), n) = sub_rat_seq(a, b, n)
    }
}

define const_rat_seq(a: Rat, n: Nat) -> Rat {
    a
}

theorem const_rat_converges_to_rat(a: Rat) {
    converges_to(lift_seq(const_rat_seq(a)), Real.from_rat(a))
} by {
    let lsa = lift_seq(const_rat_seq(a))
    forall(eps: Real) {
        if eps.is_positive {
            forall(i: Nat) {
                const_rat_seq(a, i) = a
                Real.from_rat(const_rat_seq(a, i)) = lift_seq(const_rat_seq(a), i)
                Real.from_rat(a).is_close(Real.from_rat(a), eps)
                lsa(i).is_close(Real.from_rat(a), eps)
            }
            tail_bound(lsa, Real.from_rat(a), Nat.0, eps)
        }
    }
    converges_to(lsa, Real.from_rat(a))
}

let zero_rat_seq = const_rat_seq(Rat.0)

theorem zero_rat_seq_is_zero {
    converges_to(lift_seq(zero_rat_seq), Real.0)
}

theorem neg_is_zero_sub(a: Nat -> Rat) {
    neg_rat_seq(a) = sub_rat_seq(zero_rat_seq, a)
} by {
    forall(n: Nat) {
        zero_rat_seq(n) = Rat.0
        neg_rat_seq(a, n) = -a(n)
        Rat.0 + -a(n) = -a(n)
        zero_rat_seq(n) + neg_rat_seq(a, n) = neg_rat_seq(a, n)
        add_rat_seq(zero_rat_seq, neg_rat_seq(a), n) = zero_rat_seq(n) + neg_rat_seq(a, n)
        sub_rat_seq(zero_rat_seq, a, n) = zero_rat_seq(n) - a(n)
        neg_rat_seq(a, n) = sub_rat_seq(zero_rat_seq, a, n)
    }
}

// Negating is well-defined with respect to eq_seq
theorem neg_rat_seq_well_def(a: Nat -> Rat, b: Nat -> Rat) {
    eq_seq(a, b) implies
    eq_seq(neg_rat_seq(a), neg_rat_seq(b))
} by {
    if eq_seq(a, b) {
        converges(lift_seq(a))
        converges_to(lift_seq(b), limit_rat(a))
        converges(lift_seq(b))
        converges_to(lift_seq(neg_rat_seq(a)), -limit_rat(a))
        converges(lift_seq(neg_rat_seq(a)))
        converges_to(lift_seq(neg_rat_seq(b)), -limit_rat(b))
        converges_to(lift_seq(neg_rat_seq(a)), limit(lift_seq(neg_rat_seq(a))))
        limit(lift_seq(neg_rat_seq(a))) = -limit_rat(a)
        limit(lift_seq(neg_rat_seq(a))) = limit_rat(neg_rat_seq(a))
        limit_rat(neg_rat_seq(a)) = -limit_rat(a)
        limit_rat(a) = limit_rat(b)
        -limit_rat(a) = -limit_rat(b)
        converges_to(lift_seq(neg_rat_seq(b)), -limit_rat(a))
        converges_to(lift_seq(neg_rat_seq(b)), limit_rat(neg_rat_seq(a)))
    }
}

// Addition is well-defined with respect to eq_seq
theorem add_rat_seq_well_def(a: Nat -> Rat, b: Nat -> Rat, c: Nat -> Rat, d: Nat -> Rat) {
    eq_seq(a, c) and eq_seq(b, d)
    implies
    eq_seq(add_rat_seq(a, b), add_rat_seq(c, d))
} by {
    converges_to(lift_seq(c), limit_rat(a))
    converges(lift_seq(c))
    eq_seq(d, b)
    converges(lift_seq(d))
    converges_to(lift_seq(add_rat_seq(a, b)), limit_rat(a) + limit_rat(b))
    converges_to(lift_seq(add_rat_seq(c, d)), limit_rat(c) + limit_rat(d))
    limit_rat(b) = limit_rat(d)
    limit_rat(a) + limit_rat(b) = limit_rat(c) + limit_rat(d)
    limit_rat(add_rat_seq(c, d)) = limit_rat(c) + limit_rat(d)
}

theorem sub_rat_seq_is_sub(a: Nat -> Rat, b: Nat -> Rat) {
    converges(lift_seq(a)) and converges(lift_seq(b))
    implies
    converges_to(lift_seq(sub_rat_seq(a, b)), limit_rat(a) - limit_rat(b))
} by {
    converges_to(lift_seq(neg_rat_seq(b)), -limit_rat(b))
    converges(lift_seq(neg_rat_seq(b)))
    converges_to(lift_seq(neg_rat_seq(b)), limit(lift_seq(neg_rat_seq(b))))
    limit(lift_seq(neg_rat_seq(b))) = -limit_rat(b)
    add_rat_seq(a, neg_rat_seq(b)) = sub_rat_seq(a, b)
}

// Subtraction is well-defined with respect to eq_seq
theorem sub_rat_seq_well_def(a: Nat -> Rat, b: Nat -> Rat, c: Nat -> Rat, d: Nat -> Rat) {
    eq_seq(a, c) and eq_seq(b, d)
    implies
    eq_seq(sub_rat_seq(a, b), sub_rat_seq(c, d))
} by {
    converges_to(lift_seq(c), limit_rat(a))
    converges(lift_seq(c))
    eq_seq(d, b)
    converges(lift_seq(d))
    converges_to(lift_seq(sub_rat_seq(a, b)), limit_rat(a) - limit_rat(b))
    converges_to(lift_seq(sub_rat_seq(c, d)), limit_rat(c) - limit_rat(d))
    limit_rat(b) = limit_rat(d)
    limit_rat(a) - limit_rat(b) = limit_rat(c) - limit_rat(d)
    limit_rat(sub_rat_seq(c, d)) = limit_rat(c) - limit_rat(d)
}

theorem sub_zero_imp_eq(x: Real, y: Real) {
    x - y = Real.0 implies x = y
}

// If two convergent sequences subtract to a zero-equivalent sequence, they are equal.
theorem sub_eq_zero_imp_eq(a: Nat -> Rat, b: Nat -> Rat) {
    converges(lift_seq(a)) and converges(lift_seq(b)) and eq_seq(sub_rat_seq(a, b), zero_rat_seq) implies
    eq_seq(a, b)
} by {
    if converges(lift_seq(a)) and converges(lift_seq(b)) and eq_seq(sub_rat_seq(a, b), zero_rat_seq) {
        converges_to(lift_seq(sub_rat_seq(a, b)), limit_rat(a) - limit_rat(b))
        converges(lift_seq(sub_rat_seq(a, b)))
        converges_to(lift_seq(sub_rat_seq(a, b)), limit(lift_seq(sub_rat_seq(a, b))))
        limit(lift_seq(sub_rat_seq(a, b))) = limit_rat(sub_rat_seq(a, b))
        converges_to(lift_seq(zero_rat_seq), limit_rat(sub_rat_seq(a, b)))
        converges_to(lift_seq(zero_rat_seq), Real.0)
        limit_rat(sub_rat_seq(a, b)) = Real.0
        converges_to(lift_seq(sub_rat_seq(a, b)), Real.0)
        limit_rat(a) - limit_rat(b) = Real.0
        limit_rat(a) = limit_rat(b)
    }
}

theorem self_eq(a: Nat -> Rat) {
    converges(lift_seq(a)) implies
    eq_seq(a, a)
}

// Addition of sequences is commutative because the sequences themselves are identical
theorem add_rat_seq_comm(a: Nat -> Rat, b: Nat -> Rat) {
    add_rat_seq(a, b) = add_rat_seq(b, a)
} by {
    forall(n: Nat) {
        add_rat_seq(a, b, n) = a(n) + b(n)
        add_rat_seq(b, a, n) = b(n) + a(n)
        a(n) + b(n) = b(n) + a(n)
        add_rat_seq(a, b, n) = add_rat_seq(b, a, n)
    }
}

// Multiplication of sequences is commutative because the sequences themselves are identical
theorem mul_rat_seq_comm(a: Nat -> Rat, b: Nat -> Rat) {
    mul_rat_seq(a, b) = mul_rat_seq(b, a)
} by {
    forall(n: Nat) {
        mul_rat_seq(a, b, n) = a(n) * b(n)
        mul_rat_seq(b, a, n) = b(n) * a(n)
        a(n) * b(n) = b(n) * a(n)
        mul_rat_seq(a, b, n) = mul_rat_seq(b, a, n)
    }
}

// Right-distributive property follows from sequence identity
theorem add_rat_seq_distributes_right(a: Nat -> Rat, b: Nat -> Rat, c: Nat -> Rat) {
    mul_rat_seq(a, add_rat_seq(b, c)) = add_rat_seq(mul_rat_seq(a, b), mul_rat_seq(a, c))
} by {
    forall(i: Nat) {
        mul_rat_seq(a, add_rat_seq(b, c), i) = a(i) * add_rat_seq(b, c, i)
        add_rat_seq(b, c, i) = b(i) + c(i)
        a(i) * (b(i) + c(i)) = a(i) * b(i) + a(i) * c(i)
        mul_rat_seq(a, b, i) = a(i) * b(i)
        mul_rat_seq(a, c, i) = a(i) * c(i)
        add_rat_seq(mul_rat_seq(a, b), mul_rat_seq(a, c), i) = mul_rat_seq(a, b, i) + mul_rat_seq(a, c, i)
        mul_rat_seq(a, add_rat_seq(b, c), i) = add_rat_seq(mul_rat_seq(a, b), mul_rat_seq(a, c), i)
    }
}

theorem add_rat_seq_distributes_left(a: Nat -> Rat, b: Nat -> Rat, c: Nat -> Rat) {
    mul_rat_seq(add_rat_seq(b, c), a) = add_rat_seq(mul_rat_seq(b, a), mul_rat_seq(c, a))
} by {
    forall(i: Nat) {
        mul_rat_seq(add_rat_seq(b, c), a, i) = add_rat_seq(b, c, i) * a(i)
        add_rat_seq(b, c, i) = b(i) + c(i)
        (b(i) + c(i)) * a(i) = b(i) * a(i) + c(i) * a(i)
        mul_rat_seq(b, a, i) = b(i) * a(i)
        mul_rat_seq(c, a, i) = c(i) * a(i)
        add_rat_seq(mul_rat_seq(b, a), mul_rat_seq(c, a), i) = mul_rat_seq(b, a, i) + mul_rat_seq(c, a, i)
        mul_rat_seq(add_rat_seq(b, c), a, i) = add_rat_seq(mul_rat_seq(b, a), mul_rat_seq(c, a), i)
    }
}

theorem sub_rat_seq_distributes_right(a: Nat -> Rat, b: Nat -> Rat, c: Nat -> Rat) {
    mul_rat_seq(a, sub_rat_seq(b, c)) = sub_rat_seq(mul_rat_seq(a, b), mul_rat_seq(a, c))
} by {
    forall(i: Nat) {
        mul_rat_seq(a, sub_rat_seq(b, c), i) = a(i) * sub_rat_seq(b, c, i)
        sub_rat_seq(b, c, i) = b(i) - c(i)
        a(i) * (b(i) - c(i)) = a(i) * b(i) - a(i) * c(i)
        mul_rat_seq(a, b, i) = a(i) * b(i)
        mul_rat_seq(a, c, i) = a(i) * c(i)
        sub_rat_seq(mul_rat_seq(a, b), mul_rat_seq(a, c), i) = mul_rat_seq(a, b, i) - mul_rat_seq(a, c, i)
        mul_rat_seq(a, sub_rat_seq(b, c), i) = sub_rat_seq(mul_rat_seq(a, b), mul_rat_seq(a, c), i)
    }
}

theorem sub_rat_seq_distributes_left(a: Nat -> Rat, b: Nat -> Rat, c: Nat -> Rat) {
    mul_rat_seq(sub_rat_seq(b, c), a) = sub_rat_seq(mul_rat_seq(b, a), mul_rat_seq(c, a))
} by {
    forall(i: Nat) {
        mul_rat_seq(sub_rat_seq(b, c), a, i) = sub_rat_seq(b, c, i) * a(i)
        sub_rat_seq(b, c, i) = b(i) - c(i)
        (b(i) - c(i)) * a(i) = b(i) * a(i) - c(i) * a(i)
        mul_rat_seq(b, a, i) = b(i) * a(i)
        mul_rat_seq(c, a, i) = c(i) * a(i)
        sub_rat_seq(mul_rat_seq(b, a), mul_rat_seq(c, a), i) = mul_rat_seq(b, a, i) - mul_rat_seq(c, a, i)
        mul_rat_seq(sub_rat_seq(b, c), a, i) = sub_rat_seq(mul_rat_seq(b, a), mul_rat_seq(c, a), i)
    }
}

theorem converges_to_zero_imp_eq_seq_zero(a: Nat -> Rat) {
    converges_to(lift_seq(a), Real.0)
    implies
    eq_seq(a, zero_rat_seq)
} by {
    converges(lift_seq(a))
    converges_to(lift_seq(a), limit(lift_seq(a)))
    limit(lift_seq(a)) = Real.0
    limit(lift_seq(a)) = limit_rat(a)
    limit_rat(a) = Real.0
    converges_to(lift_seq(zero_rat_seq), limit_rat(a))
}

theorem eq_seq_zero_imp_converges_to_zero(a: Nat -> Rat) {
    eq_seq(a, zero_rat_seq)
    implies
    converges_to(lift_seq(a), Real.0)
}

// Multiplying by a null sequence gives a null sequence.
theorem mul_zero_rat_seq(a: Nat -> Rat, z: Nat -> Rat) {
    converges(lift_seq(a)) and eq_seq(z, zero_rat_seq)
    implies
    eq_seq(mul_rat_seq(a, z), zero_rat_seq)
} by {
    let ub: Rat satisfy {
        forall(i: Nat) {
            a(i).abs < ub
        }
    }
    let mul_seq = lift_seq(mul_rat_seq(a, z))
    forall(eps: Real) {
        if eps.is_positive {
            let reps: Rat satisfy {
                reps.is_positive and Real.from_rat(reps) < eps
            }
            let reps2 = reps / ub
            ub.is_positive
            reps2.is_positive
            let eps2 = Real.from_rat(reps2)
            eps2.is_positive
            converges_to(lift_seq(z), Real.0)
            exists(n: Nat) {
                tail_bound(lift_seq(z), Real.0, n, eps2)
            }
            let n: Nat satisfy {
                tail_bound(lift_seq(z), Real.0, n, eps2)
            }
            forall(i: Nat) {
                if n <= i {
                    (lift_seq(z, i) + Real.0).is_close(Real.0, eps2)
                    lift_seq(z, i) + Real.0 = lift_seq(z, i)
                    lift_seq(z, i).is_close(Real.0, eps2)
                    lift_seq(z, i) = Real.from_rat(z(i))
                    Real.from_rat(z(i)).abs < eps2
                    Real.from_rat(z(i)).abs = Real.from_rat(z(i).abs)
                    Real.from_rat(z(i).abs) < eps2
                    Real.from_rat(z(i).abs) < Real.from_rat(reps2)
                    Real.from_rat(reps2) > Real.from_rat(z(i).abs)
                    z(i).abs < reps2
                    (a(i) * z(i)).abs < ub * reps2
                    ub * reps2 = reps
                    lift_seq(mul_rat_seq(a, z))(i).abs < eps
                    mul_seq(i).is_close(Real.0, eps)
                }
            }
            tail_bound(mul_seq, Real.0, n, eps)
        }
    }
    converges_to(mul_seq, Real.0)
    converges_to(lift_seq(mul_rat_seq(a, z)), Real.0)
}

theorem sub_rat_eq_imp_sub_zero(a: Nat -> Rat, b: Nat -> Rat) {
    eq_seq(a, b) implies
    eq_seq(sub_rat_seq(a, b), zero_rat_seq)
} by {
    eq_seq(b, a)
    converges(lift_seq(a))
    converges_to(lift_seq(a), limit(lift_seq(a)))
    converges_to(lift_seq(a), limit_rat(b))
    limit(lift_seq(a)) = limit_rat(b)
    limit(lift_seq(a)) = limit_rat(a)
    limit_rat(a) = limit_rat(b)
    limit_rat(a) - limit_rat(b) = Real.0
    converges_to(lift_seq(sub_rat_seq(a, b)), limit_rat(a) - limit_rat(b))
    converges_to(lift_seq(sub_rat_seq(a, b)), Real.0)
}

theorem sub_rat_sub_zero_imp_eq(a: Nat -> Rat, b: Nat -> Rat) {
    converges(lift_seq(a)) and converges(lift_seq(b)) and eq_seq(sub_rat_seq(a, b), zero_rat_seq)
    implies
    eq_seq(a, b)
}

// Prove that multiplying by a single convergent sequence is well-defined
theorem mul_right_well_def(a: Nat -> Rat, b: Nat -> Rat, c: Nat -> Rat) {
    eq_seq(b, c) and converges(lift_seq(a))
    implies
    eq_seq(mul_rat_seq(a, b), mul_rat_seq(a, c))
} by {
    converges(lift_seq(b))
    converges_to(lift_seq(c), limit_rat(b))
    converges(lift_seq(c))
    converges(lift_seq(mul_rat_seq(a, b)))
    converges(lift_seq(mul_rat_seq(a, c)))
    eq_seq(sub_rat_seq(b, c), zero_rat_seq)
    let diff: Nat -> Rat = sub_rat_seq(mul_rat_seq(a, b), mul_rat_seq(a, c))
    diff = mul_rat_seq(a, sub_rat_seq(b, c))
    eq_seq(mul_rat_seq(a, sub_rat_seq(b, c)), zero_rat_seq)
    eq_seq(diff, zero_rat_seq)
}

theorem mul_left_well_def(a: Nat -> Rat, b: Nat -> Rat, c: Nat -> Rat) {
    eq_seq(a, b) and converges(lift_seq(c))
    implies
    eq_seq(mul_rat_seq(a, c), mul_rat_seq(b, c))
} by {
    mul_rat_seq(c, a) = mul_rat_seq(a, c)
    mul_rat_seq(c, b) = mul_rat_seq(b, c)
}

theorem eq_seq_trans(a: Nat -> Rat, b: Nat -> Rat, c: Nat -> Rat) {
    eq_seq(a, b) and eq_seq(b, c)
    implies
    eq_seq(a, c)
} by {
    converges(lift_seq(a))
    converges_to(lift_seq(b), limit_rat(a))
    converges_to(lift_seq(c), limit_rat(b))
    eq_seq(c, b)
    converges_to(lift_seq(b), limit_rat(c))
    limit_rat(c) = limit_rat(a)
    converges(lift_seq(c))
    converges_to(lift_seq(a), limit(lift_seq(a)))
    limit(lift_seq(a)) = limit_rat(a)
    converges_to(lift_seq(a), limit_rat(a))
    converges_to(lift_seq(a), limit_rat(c))
    eq_seq(c, a)
}

// Prove that multiplying equivalent sequences gives equivalent things.
theorem mul_rat_eq_seq(a: Nat -> Rat, b: Nat -> Rat, c: Nat -> Rat, d: Nat -> Rat) {
    eq_seq(a, c) and eq_seq(b, d)
    implies
    eq_seq(mul_rat_seq(a, b), mul_rat_seq(c, d))
}
