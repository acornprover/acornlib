from data.basic.set import Set, all_sets_subset_universal, double_inclusion, sets_subset_union,
    subset_contains, subset_refl, subset_trans
from real.real_field import Real
from real.topology import closure, closure_mono
from real.topology_closure import closure_idempotent

/// True if a real set is dense in another real set.
define is_dense_in_real_set(s: Set[Real], t: Set[Real]) -> Bool {
    t.subset(closure(s))
}

/// True if a real set is dense in the real line.
define is_dense_real_set(s: Set[Real]) -> Bool {
    closure(s) = Set[Real].universal_set
}

/// Dense-in membership is subset membership in the closure.
theorem dense_in_real_set_contains_eq(s: Set[Real], t: Set[Real]) {
    is_dense_in_real_set(s, t) = t.subset(closure(s))
}

/// A dense set has every real number in its closure.
theorem dense_real_set_closure_contains(s: Set[Real], x: Real) {
    is_dense_real_set(s) implies closure(s).contains(x)
} by {
    if is_dense_real_set(s) {
        is_dense_real_set(s) = (closure(s) = Set[Real].universal_set)
        closure(s) = Set[Real].universal_set
        Set[Real].universal_set.contains(x)
        closure(s).contains(x)
    }
}

/// Universal closure membership makes a set dense.
theorem dense_real_set_intro(s: Set[Real]) {
    forall(x: Real) { closure(s).contains(x) } implies is_dense_real_set(s)
} by {
    if forall(x: Real) { closure(s).contains(x) } {
        all_sets_subset_universal[Real](closure(s))
        closure(s).subset(Set[Real].universal_set)
        forall(x: Real) {
            if Set[Real].universal_set.contains(x) {
                closure(s).contains(x)
            }
        }
        Set[Real].universal_set.subset(closure(s))
        double_inclusion(closure(s), Set[Real].universal_set)
        closure(s) = Set[Real].universal_set
        is_dense_real_set(s)
    }
}

/// A dense-in set carries membership of the ambient set into the closure.
theorem dense_in_real_set_contains(s: Set[Real], t: Set[Real], x: Real) {
    is_dense_in_real_set(s, t) and t.contains(x) implies closure(s).contains(x)
} by {
    if is_dense_in_real_set(s, t) and t.contains(x) {
        dense_in_real_set_contains_eq(s, t)
        t.subset(closure(s))
        subset_contains(t, closure(s), x)
        closure(s).contains(x)
    }
}

/// Every real set is dense in its own closure.
theorem dense_in_closure_self(s: Set[Real]) {
    is_dense_in_real_set(s, closure(s))
} by {
    subset_refl[Real](closure(s))
    closure(s).subset(closure(s))
    is_dense_in_real_set(s, closure(s))
}

/// A real set is dense in every subset of its closure.
theorem dense_in_subset_of_closure(s: Set[Real], t: Set[Real]) {
    t.subset(closure(s)) implies is_dense_in_real_set(s, t)
}

/// Dense-in is preserved by enlarging the dense subset.
theorem dense_in_of_subset_left(s: Set[Real], t: Set[Real], u: Set[Real]) {
    s.subset(t) and is_dense_in_real_set(s, u) implies is_dense_in_real_set(t, u)
} by {
    if s.subset(t) and is_dense_in_real_set(s, u) {
        closure_mono(s, t)
        closure(s).subset(closure(t))
        u.subset(closure(s))
        subset_trans[Real](u, closure(s), closure(t))
        u.subset(closure(t))
        is_dense_in_real_set(t, u)
    }
}

/// Dense-in is preserved by shrinking the ambient set.
theorem dense_in_of_subset_right(s: Set[Real], t: Set[Real], u: Set[Real]) {
    u.subset(t) and is_dense_in_real_set(s, t) implies is_dense_in_real_set(s, u)
} by {
    if u.subset(t) and is_dense_in_real_set(s, t) {
        t.subset(closure(s))
        subset_trans[Real](u, t, closure(s))
        u.subset(closure(s))
        is_dense_in_real_set(s, u)
    }
}

/// Dense-in is transitive.
theorem dense_in_trans(s: Set[Real], t: Set[Real], u: Set[Real]) {
    is_dense_in_real_set(s, t) and is_dense_in_real_set(t, u) implies is_dense_in_real_set(s, u)
} by {
    if is_dense_in_real_set(s, t) and is_dense_in_real_set(t, u) {
        t.subset(closure(s))
        u.subset(closure(t))
        closure_mono(t, closure(s))
        closure(t).subset(closure(closure(s)))
        closure_idempotent(s)
        closure(closure(s)) = closure(s)
        closure(t).subset(closure(s))
        subset_trans[Real](u, closure(t), closure(s))
        u.subset(closure(s))
        is_dense_in_real_set(s, u)
    }
}

/// Enlarging a dense real set preserves density.
theorem dense_real_set_of_subset(s: Set[Real], t: Set[Real]) {
    s.subset(t) and is_dense_real_set(s) implies is_dense_real_set(t)
} by {
    if s.subset(t) and is_dense_real_set(s) {
        closure_mono(s, t)
        closure(s).subset(closure(t))
        closure(s) = Set[Real].universal_set
        Set[Real].universal_set.subset(closure(t))
        all_sets_subset_universal[Real](closure(t))
        closure(t).subset(Set[Real].universal_set)
        double_inclusion(closure(t), Set[Real].universal_set)
        closure(t) = Set[Real].universal_set
        is_dense_real_set(t)
    }
}

/// The universal real set is dense.
theorem universal_real_set_is_dense {
    is_dense_real_set(Set[Real].universal_set)
} by {
    forall(x: Real) {
        Set[Real].universal_set.contains(x)
        closure(Set[Real].universal_set).contains(x)
    }
    dense_real_set_intro(Set[Real].universal_set)
}

/// A dense real set is dense in every real set.
theorem dense_real_set_dense_in_any(s: Set[Real], t: Set[Real]) {
    is_dense_real_set(s) implies is_dense_in_real_set(s, t)
} by {
    if is_dense_real_set(s) {
        closure(s) = Set[Real].universal_set
        all_sets_subset_universal[Real](t)
        t.subset(Set[Real].universal_set)
        t.subset(closure(s))
        is_dense_in_real_set(s, t)
    }
}

/// A set dense in the real line is dense as a real set.
theorem dense_real_set_of_dense_in_universal(s: Set[Real]) {
    is_dense_in_real_set(s, Set[Real].universal_set) implies is_dense_real_set(s)
} by {
    if is_dense_in_real_set(s, Set[Real].universal_set) {
        Set[Real].universal_set.subset(closure(s))
        all_sets_subset_universal[Real](closure(s))
        closure(s).subset(Set[Real].universal_set)
        double_inclusion(closure(s), Set[Real].universal_set)
        closure(s) = Set[Real].universal_set
        is_dense_real_set(s)
    }
}

/// Density in the real line is the same as density as a real set.
theorem dense_in_universal_eq_dense_real_set(s: Set[Real]) {
    is_dense_in_real_set(s, Set[Real].universal_set) = is_dense_real_set(s)
} by {
    if is_dense_in_real_set(s, Set[Real].universal_set) {
        dense_real_set_of_dense_in_universal(s)
        is_dense_real_set(s)
    }
    if is_dense_real_set(s) {
        dense_real_set_dense_in_any(s, Set[Real].universal_set)
        is_dense_in_real_set(s, Set[Real].universal_set)
    }
    is_dense_in_real_set(s, Set[Real].universal_set) = is_dense_real_set(s)
}

/// A set is dense exactly when its closure is dense.
theorem dense_closure_eq_dense(s: Set[Real]) {
    is_dense_real_set(closure(s)) = is_dense_real_set(s)
} by {
    closure_idempotent(s)
    closure(closure(s)) = closure(s)
    if is_dense_real_set(closure(s)) {
        is_dense_real_set(closure(s)) = (closure(closure(s)) = Set[Real].universal_set)
        closure(closure(s)) = Set[Real].universal_set
        closure(s) = Set[Real].universal_set
        is_dense_real_set(s)
    }
    if is_dense_real_set(s) {
        closure(s) = Set[Real].universal_set
        closure(closure(s)) = Set[Real].universal_set
        is_dense_real_set(closure(s))
    }
    is_dense_real_set(closure(s)) = is_dense_real_set(s)
}

/// A set is dense in an ambient set exactly when its closure is dense there.
theorem dense_in_closure_left_eq_dense_in(s: Set[Real], t: Set[Real]) {
    is_dense_in_real_set(closure(s), t) = is_dense_in_real_set(s, t)
} by {
    closure_idempotent(s)
    closure(closure(s)) = closure(s)
    if is_dense_in_real_set(closure(s), t) {
        t.subset(closure(closure(s)))
        t.subset(closure(s))
        is_dense_in_real_set(s, t)
    }
    if is_dense_in_real_set(s, t) {
        t.subset(closure(s))
        t.subset(closure(closure(s)))
        is_dense_in_real_set(closure(s), t)
    }
    is_dense_in_real_set(closure(s), t) = is_dense_in_real_set(s, t)
}

/// Taking a union on the right preserves density of the left member.
theorem dense_union_left(s: Set[Real], t: Set[Real]) {
    is_dense_real_set(s) implies is_dense_real_set(s.union(t))
} by {
    if is_dense_real_set(s) {
        sets_subset_union[Real](s, t)
        s.subset(s.union(t))
        dense_real_set_of_subset(s, s.union(t))
        is_dense_real_set(s.union(t))
    }
}

/// Taking a union on the left preserves density of the right member.
theorem dense_union_right(s: Set[Real], t: Set[Real]) {
    is_dense_real_set(t) implies is_dense_real_set(s.union(t))
} by {
    if is_dense_real_set(t) {
        sets_subset_union[Real](s, t)
        t.subset(s.union(t))
        dense_real_set_of_subset(t, s.union(t))
        is_dense_real_set(s.union(t))
    }
}

/// Enlarging by a union preserves dense-in membership on the left.
theorem dense_in_union_left(s: Set[Real], t: Set[Real], u: Set[Real]) {
    is_dense_in_real_set(s, u) implies is_dense_in_real_set(s.union(t), u)
} by {
    if is_dense_in_real_set(s, u) {
        sets_subset_union[Real](s, t)
        s.subset(s.union(t))
        dense_in_of_subset_left(s, s.union(t), u)
        is_dense_in_real_set(s.union(t), u)
    }
}

/// Enlarging by a union preserves dense-in membership on the right.
theorem dense_in_union_right(s: Set[Real], t: Set[Real], u: Set[Real]) {
    is_dense_in_real_set(t, u) implies is_dense_in_real_set(s.union(t), u)
} by {
    if is_dense_in_real_set(t, u) {
        sets_subset_union[Real](s, t)
        t.subset(s.union(t))
        dense_in_of_subset_left(t, s.union(t), u)
        is_dense_in_real_set(s.union(t), u)
    }
}

/// If a set is dense in an ambient set, its closure is also dense in that ambient set.
theorem closure_dense_in_of_dense_in(s: Set[Real], t: Set[Real]) {
    is_dense_in_real_set(s, t) implies is_dense_in_real_set(closure(s), t)
} by {
    if is_dense_in_real_set(s, t) {
        dense_in_closure_left_eq_dense_in(s, t)
        is_dense_in_real_set(closure(s), t) = is_dense_in_real_set(s, t)
        is_dense_in_real_set(closure(s), t)
    }
}

/// If a real set is dense, its closure is dense.
theorem closure_of_dense_real_set_is_dense(s: Set[Real]) {
    is_dense_real_set(s) implies is_dense_real_set(closure(s))
} by {
    if is_dense_real_set(s) {
        dense_closure_eq_dense(s)
        is_dense_real_set(closure(s)) = is_dense_real_set(s)
        is_dense_real_set(closure(s))
    }
}
