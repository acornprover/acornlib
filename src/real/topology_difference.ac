from data.basic.set import Set, difference_contains_intro, difference_contains_not_right, difference_subset
from real.real_field import Real
from real.topology import adherent_point_eps, adherent_point_intro, closed_set_contains_adherent,
    eps_adherent_of_subset, is_adherent_point_of_set, is_eps_adherent_to_set,
    is_closed_set, is_interior_point, is_open_set, open_set_interior_point

/// Adherence to a set difference gives adherence to the left set.
theorem adherent_difference_left(s: Set[Real], t: Set[Real], x: Real) {
    is_adherent_point_of_set(s.difference(t), x) implies is_adherent_point_of_set(s, x)
} by {
    if is_adherent_point_of_set(s.difference(t), x) {
        difference_subset[Real](s, t)
        s.difference(t).subset(s)
        forall(eps: Real) {
            if eps.is_positive {
                adherent_point_eps(s.difference(t), x, eps)
                is_eps_adherent_to_set(s.difference(t), x, eps)
                eps_adherent_of_subset(s.difference(t), s, x, eps)
                is_eps_adherent_to_set(s, x, eps)
            }
        }
        adherent_point_intro(s, x)
        is_adherent_point_of_set(s, x)
    }
}

/// An adherent point of a difference cannot lie in an open right-hand set.
theorem adherent_difference_not_open_right(s: Set[Real], t: Set[Real], x: Real) {
    is_open_set(t) and is_adherent_point_of_set(s.difference(t), x) implies not t.contains(x)
} by {
    if is_open_set(t) and is_adherent_point_of_set(s.difference(t), x) {
        if t.contains(x) {
            open_set_interior_point(t, x)
            is_interior_point(t, x)
            let eps: Real satisfy {
                eps.is_positive and forall(y: Real) {
                    y.is_close(x, eps) implies t.contains(y)
                }
            }
            adherent_point_eps(s.difference(t), x, eps)
            is_eps_adherent_to_set(s.difference(t), x, eps)
            let y: Real satisfy {
                s.difference(t).contains(y) and y.is_close(x, eps)
            }
            t.contains(y)
            difference_contains_not_right[Real](s, t, y)
            not t.contains(y)
            false
        }
    }
}

/// Removing an open real set from a closed real set preserves closedness.
theorem difference_of_closed_and_open_is_closed(s: Set[Real], t: Set[Real]) {
    is_closed_set(s) and is_open_set(t) implies is_closed_set(s.difference(t))
} by {
    if is_closed_set(s) and is_open_set(t) {
        forall(x: Real) {
            if is_adherent_point_of_set(s.difference(t), x) {
                adherent_difference_left(s, t, x)
                is_adherent_point_of_set(s, x)
                closed_set_contains_adherent(s, x)
                s.contains(x)
                adherent_difference_not_open_right(s, t, x)
                not t.contains(x)
                difference_contains_intro[Real](s, t, x)
                s.difference(t).contains(x)
            }
        }
        is_closed_set(s.difference(t))
    }
}
