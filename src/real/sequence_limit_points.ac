from nat import Nat
from order import max_imp_gte
from real.real_field import Real
from real.real_seq import converges_to, tail_bound, tail_bound_implies_is_close

/// True if `u` is `eps`-adherent to the tail of `a` starting at `m`.
define is_eps_adherent_from(a: Nat -> Real, m: Nat, u: Real, eps: Real) -> Bool {
    forall(n_start: Nat) {
        m <= n_start implies exists(n_witness: Nat) {
            n_start <= n_witness and a(n_witness).is_close(u, eps)
        }
    }
}

/// True if `u` is a limit point of the tail of `a` starting at `m`.
define is_limit_point_from(a: Nat -> Real, m: Nat, u: Real) -> Bool {
    forall(eps: Real) {
        eps.is_positive implies is_eps_adherent_from(a, m, u, eps)
    }
}

/// A convergent sequence has a tail bound for every positive epsilon.
theorem converges_to_has_tail_bound(a: Nat -> Real, u: Real, eps: Real) {
    converges_to(a, u) and eps.is_positive implies exists(n: Nat) {
        tail_bound(a, u, n, eps)
    }
} by {
    if converges_to(a, u) and eps.is_positive {
        converges_to(a, u) = forall(e: Real) {
            e.is_positive implies exists(n: Nat) {
                tail_bound(a, u, n, e)
            }
        }
        forall(e: Real) {
            e.is_positive implies exists(n: Nat) {
                tail_bound(a, u, n, e)
            }
        }
        exists(n: Nat) {
            tail_bound(a, u, n, eps)
        }
    }
}

/// A convergence tail bound gives an epsilon-adherent witness from any later start.
theorem tail_bound_imp_eps_adherent_from(a: Nat -> Real, m: Nat, u: Real, eps: Real, n0: Nat) {
    tail_bound(a, u, n0, eps) implies is_eps_adherent_from(a, m, u, eps)
} by {
    if tail_bound(a, u, n0, eps) {
        forall(n_start: Nat) {
            if m <= n_start {
                let n = n_start.max(n0)
                max_imp_gte[Nat](n_start, n0)
                n >= n_start and n >= n0
                n_start <= n
                n0 <= n
                tail_bound_implies_is_close(a, u, n0, eps, n)
                a(n).is_close(u, eps)
                exists(n_witness: Nat) {
                    n_start <= n_witness and a(n_witness).is_close(u, eps)
                }
            }
        }
        is_eps_adherent_from(a, m, u, eps) = forall(n_start: Nat) {
            m <= n_start implies exists(n_witness: Nat) {
                n_start <= n_witness and a(n_witness).is_close(u, eps)
            }
        }
        is_eps_adherent_from(a, m, u, eps)
    }
}

/// A convergent sequence has its limit as a limit point of every tail.
theorem converges_to_imp_limit_point_from(a: Nat -> Real, m: Nat, u: Real) {
    converges_to(a, u) implies is_limit_point_from(a, m, u)
} by {
    if converges_to(a, u) {
        forall(eps: Real) {
            if eps.is_positive {
                converges_to_has_tail_bound(a, u, eps)
                let n0: Nat satisfy {
                    tail_bound(a, u, n0, eps)
                }
                tail_bound_imp_eps_adherent_from(a, m, u, eps, n0)
                is_eps_adherent_from(a, m, u, eps)
            }
        }
        is_limit_point_from(a, m, u)
    }
}
