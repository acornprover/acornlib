/// Limit algebra for real functions: the limit laws (sum, product, quotient,
/// composition) expressed in the library's sequential limit form `limit_at`:
/// a function f has limit l at a if every sequence converging to a with terms
/// distinct from a is carried by f to a sequence converging to l.
///
/// The sequence-level laws (`limit_add_seq`, `limit_prod_seq`,
/// `converges_to_div_seq`) already exist in the sequence modules; this file
/// lifts them to functions and adds the composition law.

from data.basic.function_algebra import pointwise_mul
from data.basic.functions import compose
from nat import Nat
from real.continuity_base import Real, add_fns, continuous_at
from real.continuity_sequences import continuous_at_preserves_sequence_limit
from real.lhopital import limit_at, quotient_fn, div_seq, converges_to_div_seq,
    seq_pointwise_eq_converges_to
from real.prod_seq import prod_seq, limit_prod_seq
from real.real_base import self_close
from real.real_seq import converges, converges_to, limit, add_seq, tail_bound,
    converges_to_imp_converges, converges_imp_converges_to, converges_to_unique,
    limit_add_seq

/// Applying a limit law to a sequence: if f has limit l at a and q converges
/// to a with terms distinct from a, then compose(f, q) converges to l.
theorem limit_at_seq(f: Real -> Real, a: Real, l: Real, q: Nat -> Real) {
    limit_at(f, a, l) and (forall(n: Nat) { q(n) != a }) and converges_to(q, a)
    implies converges_to(compose(f, q), l)
} by {
    if limit_at(f, a, l) and (forall(n: Nat) { q(n) != a }) and converges_to(q, a) {
        limit_at(f, a, l) = forall(q1: Nat -> Real) {
            (forall(n: Nat) { q1(n) != a }) and converges_to(q1, a)
            implies converges_to(compose(f, q1), l)
        }
        converges_to(compose(f, q), l)
    }
}

/// The sum of two sequences converging to given limits converges to the sum of
/// those limits.
theorem converges_to_add_seq(a: Nat -> Real, b: Nat -> Real, a_lim: Real, b_lim: Real) {
    converges_to(a, a_lim) and converges_to(b, b_lim)
    implies converges_to(add_seq(a, b), a_lim + b_lim)
} by {
    if converges_to(a, a_lim) and converges_to(b, b_lim) {
        converges_to_imp_converges(a, a_lim)
        converges(a)
        converges_to_imp_converges(b, b_lim)
        converges(b)
        limit_add_seq(a, b)
        converges_to(add_seq(a, b), limit(a) + limit(b))
        converges_imp_converges_to(a)
        converges_to(a, limit(a))
        converges_to_unique(a, a_lim, limit(a))
        a_lim = limit(a)
        limit(a) = a_lim
        converges_imp_converges_to(b)
        converges_to(b, limit(b))
        converges_to_unique(b, b_lim, limit(b))
        b_lim = limit(b)
        limit(b) = b_lim
        limit(a) + limit(b) = a_lim + b_lim
        converges_to(add_seq(a, b), a_lim + b_lim)
    }
}

/// The product of two sequences converging to given limits converges to the
/// product of those limits.
theorem converges_to_prod_seq(a: Nat -> Real, b: Nat -> Real, a_lim: Real, b_lim: Real) {
    converges_to(a, a_lim) and converges_to(b, b_lim)
    implies converges_to(prod_seq(a, b), a_lim * b_lim)
} by {
    if converges_to(a, a_lim) and converges_to(b, b_lim) {
        converges_to_imp_converges(a, a_lim)
        converges(a)
        converges_to_imp_converges(b, b_lim)
        converges(b)
        limit_prod_seq(a, b)
        converges_to(prod_seq(a, b), limit(a) * limit(b))
        converges_imp_converges_to(a)
        converges_to(a, limit(a))
        converges_to_unique(a, a_lim, limit(a))
        a_lim = limit(a)
        limit(a) = a_lim
        converges_imp_converges_to(b)
        converges_to(b, limit(b))
        converges_to_unique(b, b_lim, limit(b))
        b_lim = limit(b)
        limit(b) = b_lim
        limit(a) * limit(b) = a_lim * b_lim
        converges_to(prod_seq(a, b), a_lim * b_lim)
    }
}

/// A function continuous at a has the limit f(a) at a.
theorem continuous_at_imp_limit_at(f: Real -> Real, a: Real) {
    continuous_at(f, a) implies limit_at(f, a, f(a))
} by {
    if continuous_at(f, a) {
        forall(q: Nat -> Real) {
            if (forall(n: Nat) { q(n) != a }) and converges_to(q, a) {
                continuous_at_preserves_sequence_limit(f, q, a)
                converges_to(compose(f, q), f(a))
            }
        }
    }
}

/// The limit of the pointwise sum of two functions is the sum of their limits.
theorem limit_at_add_fns(f: Real -> Real, g: Real -> Real, a: Real, l: Real, m: Real) {
    limit_at(f, a, l) and limit_at(g, a, m)
    implies limit_at(add_fns(f, g), a, l + m)
} by {
    if limit_at(f, a, l) and limit_at(g, a, m) {
        forall(q: Nat -> Real) {
            if (forall(n: Nat) { q(n) != a }) and converges_to(q, a) {
                limit_at_seq(f, a, l, q)
                converges_to(compose(f, q), l)
                limit_at_seq(g, a, m, q)
                converges_to(compose(g, q), m)
                converges_to_add_seq(compose(f, q), compose(g, q), l, m)
                converges_to(add_seq(compose(f, q), compose(g, q)), l + m)
                forall(n: Nat) {
                    compose(add_fns(f, g), q, n) = add_fns(f, g, q(n))
                    add_fns(f, g, q(n)) = f(q(n)) + g(q(n))
                    compose(f, q, n) = f(q(n))
                    compose(g, q, n) = g(q(n))
                    add_seq(compose(f, q), compose(g, q), n) =
                        compose(f, q, n) + compose(g, q, n)
                    add_seq(compose(f, q), compose(g, q), n) = f(q(n)) + g(q(n))
                    compose(add_fns(f, g), q, n) =
                        add_seq(compose(f, q), compose(g, q), n)
                }
                seq_pointwise_eq_converges_to(
                    add_seq(compose(f, q), compose(g, q)),
                    compose(add_fns(f, g), q), l + m)
                converges_to(compose(add_fns(f, g), q), l + m)
            }
        }
    }
}

/// The limit of the pointwise product of two functions is the product of their
/// limits.
theorem limit_at_pointwise_mul(f: Real -> Real, g: Real -> Real, a: Real, l: Real, m: Real) {
    limit_at(f, a, l) and limit_at(g, a, m)
    implies limit_at(pointwise_mul[Real, Real](f, g), a, l * m)
} by {
    if limit_at(f, a, l) and limit_at(g, a, m) {
        forall(q: Nat -> Real) {
            if (forall(n: Nat) { q(n) != a }) and converges_to(q, a) {
                limit_at_seq(f, a, l, q)
                converges_to(compose(f, q), l)
                limit_at_seq(g, a, m, q)
                converges_to(compose(g, q), m)
                converges_to_prod_seq(compose(f, q), compose(g, q), l, m)
                converges_to(prod_seq(compose(f, q), compose(g, q)), l * m)
                forall(n: Nat) {
                    prod_seq(compose(f, q), compose(g, q), n) =
                        compose(f, q, n) * compose(g, q, n)
                    compose(f, q, n) = f(q(n))
                    compose(g, q, n) = g(q(n))
                    prod_seq(compose(f, q), compose(g, q), n) = f(q(n)) * g(q(n))
                    pointwise_mul[Real, Real](f, g, q(n)) = f(q(n)) * g(q(n))
                    prod_seq(compose(f, q), compose(g, q), n) =
                        pointwise_mul[Real, Real](f, g, q(n))
                    compose(pointwise_mul[Real, Real](f, g), q, n) =
                        pointwise_mul[Real, Real](f, g, q(n))
                    prod_seq(compose(f, q), compose(g, q), n) =
                        compose(pointwise_mul[Real, Real](f, g), q, n)
                }
                seq_pointwise_eq_converges_to(
                    prod_seq(compose(f, q), compose(g, q)),
                    compose(pointwise_mul[Real, Real](f, g), q), l * m)
                converges_to(compose(pointwise_mul[Real, Real](f, g), q), l * m)
            }
        }
    }
}

/// The limit of the pointwise quotient of two functions is the quotient of
/// their limits, provided the limit of the denominator is nonzero.
theorem limit_at_quotient_fn(f: Real -> Real, g: Real -> Real, a: Real, l: Real, m: Real) {
    limit_at(f, a, l) and limit_at(g, a, m) and m != Real.0
    implies limit_at(quotient_fn(f, g), a, l / m)
} by {
    if limit_at(f, a, l) and limit_at(g, a, m) and m != Real.0 {
        forall(q: Nat -> Real) {
            if (forall(n: Nat) { q(n) != a }) and converges_to(q, a) {
                limit_at_seq(f, a, l, q)
                converges_to(compose(f, q), l)
                limit_at_seq(g, a, m, q)
                converges_to(compose(g, q), m)
                converges_to_div_seq(compose(f, q), compose(g, q), l, m)
                converges_to(div_seq(compose(f, q), compose(g, q)), l / m)
                forall(n: Nat) {
                    div_seq(compose(f, q), compose(g, q), n) =
                        compose(f, q, n) / compose(g, q, n)
                    compose(f, q, n) = f(q(n))
                    compose(g, q, n) = g(q(n))
                    div_seq(compose(f, q), compose(g, q), n) = f(q(n)) / g(q(n))
                    quotient_fn(f, g, q(n)) = f(q(n)) / g(q(n))
                    div_seq(compose(f, q), compose(g, q), n) = quotient_fn(f, g, q(n))
                    compose(quotient_fn(f, g), q, n) = quotient_fn(f, g, q(n))
                    div_seq(compose(f, q), compose(g, q), n) =
                        compose(quotient_fn(f, g), q, n)
                }
                seq_pointwise_eq_converges_to(
                    div_seq(compose(f, q), compose(g, q)),
                    compose(quotient_fn(f, g), q), l / m)
                converges_to(compose(quotient_fn(f, g), q), l / m)
            }
        }
    }
}

/// The composition law: if f tends to b at a and g is continuous at b, then
/// g composed with f tends to g(b) at a.
theorem limit_at_compose_continuous(f: Real -> Real, g: Real -> Real, a: Real, b: Real) {
    limit_at(f, a, b) and continuous_at(g, b)
    implies limit_at(compose(g, f), a, g(b))
} by {
    if limit_at(f, a, b) and continuous_at(g, b) {
        forall(q: Nat -> Real) {
            if (forall(n: Nat) { q(n) != a }) and converges_to(q, a) {
                limit_at_seq(f, a, b, q)
                converges_to(compose(f, q), b)
                continuous_at_preserves_sequence_limit(g, compose(f, q), b)
                converges_to(compose(g, compose(f, q)), g(b))
                forall(n: Nat) {
                    compose(g, compose(f, q), n) = g(compose(f, q, n))
                    compose(f, q, n) = f(q(n))
                    g(compose(f, q, n)) = g(f(q(n)))
                    compose(g, f, q(n)) = g(f(q(n)))
                    compose(compose(g, f), q, n) = compose(g, f, q(n))
                    compose(g, compose(f, q), n) = compose(compose(g, f), q, n)
                }
                seq_pointwise_eq_converges_to(
                    compose(g, compose(f, q)),
                    compose(compose(g, f), q), g(b))
                converges_to(compose(compose(g, f), q), g(b))
            }
        }
    }
}

/// The composition law in limit form: if f tends to b at a, f avoids b away
/// from a, and g tends to m at b, then g composed with f tends to m at a.
theorem limit_at_compose(f: Real -> Real, g: Real -> Real, a: Real, b: Real, m: Real) {
    limit_at(f, a, b) and limit_at(g, b, m) and
    (forall(x: Real) { x != a implies f(x) != b })
    implies limit_at(compose(g, f), a, m)
} by {
    if limit_at(f, a, b) and limit_at(g, b, m) and
       (forall(x: Real) { x != a implies f(x) != b }) {
        forall(q: Nat -> Real) {
            if (forall(n: Nat) { q(n) != a }) and converges_to(q, a) {
                limit_at_seq(f, a, b, q)
                converges_to(compose(f, q), b)
                forall(n: Nat) {
                    compose(f, q, n) = f(q(n))
                    q(n) != a
                    forall(x: Real) { x != a implies f(x) != b }
                    q(n) != a implies f(q(n)) != b
                    f(q(n)) != b
                    compose(f, q, n) != b
                }
                limit_at_seq(g, b, m, compose(f, q))
                converges_to(compose(g, compose(f, q)), m)
                forall(n: Nat) {
                    compose(g, compose(f, q), n) = g(compose(f, q, n))
                    compose(f, q, n) = f(q(n))
                    g(compose(f, q, n)) = g(f(q(n)))
                    compose(g, f, q(n)) = g(f(q(n)))
                    compose(compose(g, f), q, n) = compose(g, f, q(n))
                    compose(g, compose(f, q), n) = compose(compose(g, f), q, n)
                }
                seq_pointwise_eq_converges_to(
                    compose(g, compose(f, q)),
                    compose(compose(g, f), q), m)
                converges_to(compose(compose(g, f), q), m)
            }
        }
    }
}

/// A constant function has the constant value as its limit at every point.
theorem limit_at_const(c: Real, a: Real) {
    limit_at(constant[Real, Real](c), a, c)
} by {
    forall(q: Nat -> Real) {
        if (forall(n: Nat) { q(n) != a }) and converges_to(q, a) {
            forall(eps: Real) {
                if eps.is_positive {
                    self_close(c, eps)
                    forall(i: Nat) {
                        if Nat.0 <= i {
                            compose(constant[Real, Real](c), q, i) =
                                constant[Real, Real](c, q(i))
                            constant[Real, Real](c, q(i)) = c
                            compose(constant[Real, Real](c), q, i) = c
                            c.is_close(c, eps)
                            compose(constant[Real, Real](c), q, i).is_close(c, eps)
                        }
                    }
                    tail_bound(compose(constant[Real, Real](c), q), c, Nat.0, eps)
                    exists(n: Nat) {
                        tail_bound(compose(constant[Real, Real](c), q), c, n, eps)
                    }
                }
            }
            converges_to(compose(constant[Real, Real](c), q), c)
        }
    }
}

/// Multiplying a function by a constant scales its limit.
theorem limit_at_const_mul(f: Real -> Real, a: Real, l: Real, c: Real) {
    limit_at(f, a, l)
    implies limit_at(pointwise_mul[Real, Real](constant[Real, Real](c), f), a, c * l)
} by {
    if limit_at(f, a, l) {
        limit_at_const(c, a)
        limit_at_pointwise_mul(constant[Real, Real](c), f, a, c, l)
        limit_at(pointwise_mul[Real, Real](constant[Real, Real](c), f), a, c * l)
    }
}
