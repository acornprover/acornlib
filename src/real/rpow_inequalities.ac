from order import lt_of_lt_of_lte
from real.exp_inequalities import exp_ge_one_of_nonneg, exp_le_one_of_nonpos, exp_monotone
from real.log import Real, log_some_of_pos_exists
from real.log_inequalities import log_nonneg_of_ge_one, log_nonpos_of_pos_le_one
from real.real_ring import lte_mul_nonneg_left, lte_mul_nonneg_right, mul_neg_left, mul_neg_right, mul_nonneg, mul_zero_left

numerals Real

/// A base at least one raised to a nonnegative real exponent is at least one.
theorem rpow_ge_one_of_base_ge_one_exp_nonneg(base: Real, exponent: Real) {
    base >= Real.1 and exponent >= Real.0 implies exists(value: Real) {
        base.rpow(exponent) = Option.some(value) and value >= Real.1
    }
} by {
    Real.1 > Real.0
    lt_of_lt_of_lte(Real.0, Real.1, base)
    Real.0 < base
    base > Real.0
    log_some_of_pos_exists(base)
    let lb: Real satisfy {
        base.log = Option.some(lb)
    }
    option_get_or_else_some[Real](lb, Real.0)
    option_get_or_else(Option.some(lb), Real.0) = lb
    base.log.get_or_else(Real.0) = lb
    base.log = Option.some(base.log.get_or_else(Real.0))
    log_nonneg_of_ge_one(base, base.log.get_or_else(Real.0))
    base.log.get_or_else(Real.0) >= Real.0
    exponent * base.log.get_or_else(Real.0) >= Real.0
    base.rpow(exponent) = Option.some((exponent * base.log.get_or_else(Real.0)).exp)
    exp_ge_one_of_nonneg(exponent * base.log.get_or_else(Real.0))
    (exponent * base.log.get_or_else(Real.0)).exp >= Real.1
    exists(value: Real) {
        value = (exponent * base.log.get_or_else(Real.0)).exp and
        base.rpow(exponent) = Option.some(value) and value >= Real.1
    }
}

/// A base at least one raised to a nonpositive real exponent is at most one.
theorem rpow_le_one_of_base_ge_one_exp_nonpos(base: Real, exponent: Real) {
    base >= Real.1 and exponent <= Real.0 implies exists(value: Real) {
        base.rpow(exponent) = Option.some(value) and value <= Real.1
    }
} by {
    Real.1 > Real.0
    lt_of_lt_of_lte(Real.0, Real.1, base)
    Real.0 < base
    base > Real.0
    log_some_of_pos_exists(base)
    let lb: Real satisfy {
        base.log = Option.some(lb)
    }
    option_get_or_else_some[Real](lb, Real.0)
    option_get_or_else(Option.some(lb), Real.0) = lb
    base.log.get_or_else(Real.0) = lb
    base.log = Option.some(base.log.get_or_else(Real.0))
    log_nonneg_of_ge_one(base, base.log.get_or_else(Real.0))
    base.log.get_or_else(Real.0) >= Real.0
    not base.log.get_or_else(Real.0).is_negative
    lte_mul_nonneg_right(exponent, Real.0, base.log.get_or_else(Real.0))
    exponent * base.log.get_or_else(Real.0) <= Real.0 * base.log.get_or_else(Real.0)
    mul_zero_left(base.log.get_or_else(Real.0))
    Real.0 * base.log.get_or_else(Real.0) = Real.0
    exponent * base.log.get_or_else(Real.0) <= Real.0
    base.rpow(exponent) = Option.some((exponent * base.log.get_or_else(Real.0)).exp)
    exp_le_one_of_nonpos(exponent * base.log.get_or_else(Real.0))
    (exponent * base.log.get_or_else(Real.0)).exp <= Real.1
    exists(value: Real) {
        value = (exponent * base.log.get_or_else(Real.0)).exp and
        base.rpow(exponent) = Option.some(value) and value <= Real.1
    }
}

/// A positive base at most one raised to a nonnegative real exponent is at most one.
theorem rpow_le_one_of_pos_base_le_one_exp_nonneg(base: Real, exponent: Real) {
    base > Real.0 and base <= Real.1 and exponent >= Real.0 implies exists(value: Real) {
        base.rpow(exponent) = Option.some(value) and value <= Real.1
    }
} by {
    log_some_of_pos_exists(base)
    let lb: Real satisfy {
        base.log = Option.some(lb)
    }
    option_get_or_else_some[Real](lb, Real.0)
    option_get_or_else(Option.some(lb), Real.0) = lb
    base.log.get_or_else(Real.0) = lb
    log_nonpos_of_pos_le_one(base, lb)
    lb <= Real.0
    not exponent.is_negative
    lte_mul_nonneg_left(lb, Real.0, exponent)
    exponent * lb <= exponent * Real.0
    exponent * Real.0 = Real.0
    exponent * lb <= Real.0
    base.rpow(exponent) = Option.some((exponent * lb).exp)
    exp_le_one_of_nonpos(exponent * lb)
    (exponent * lb).exp <= Real.1
    exists(value: Real) {
        value = (exponent * base.log.get_or_else(Real.0)).exp and
        base.rpow(exponent) = Option.some(value) and value <= Real.1
    }
}

/// A positive base at most one raised to a nonpositive real exponent is at least one.
theorem rpow_ge_one_of_pos_base_le_one_exp_nonpos(base: Real, exponent: Real) {
    base > Real.0 and base <= Real.1 and exponent <= Real.0 implies exists(value: Real) {
        base.rpow(exponent) = Option.some(value) and value >= Real.1
    }
} by {
    log_some_of_pos_exists(base)
    let lb: Real satisfy {
        base.log = Option.some(lb)
    }
    option_get_or_else_some[Real](lb, Real.0)
    option_get_or_else(Option.some(lb), Real.0) = lb
    base.log.get_or_else(Real.0) = lb
    base.log = Option.some(base.log.get_or_else(Real.0))
    log_nonpos_of_pos_le_one(base, base.log.get_or_else(Real.0))
    base.log.get_or_else(Real.0) <= Real.0
    -exponent >= Real.0
    -base.log.get_or_else(Real.0) >= Real.0
    mul_nonneg(-exponent, -base.log.get_or_else(Real.0))
    (-exponent) * (-base.log.get_or_else(Real.0)) >= Real.0
    mul_neg_left(exponent, -base.log.get_or_else(Real.0))
    (-exponent) * (-base.log.get_or_else(Real.0)) = -(exponent * -base.log.get_or_else(Real.0))
    mul_neg_right(exponent, base.log.get_or_else(Real.0))
    exponent * -base.log.get_or_else(Real.0) = -(exponent * base.log.get_or_else(Real.0))
    -(exponent * -base.log.get_or_else(Real.0)) = --(exponent * base.log.get_or_else(Real.0))
    --(exponent * base.log.get_or_else(Real.0)) = exponent * base.log.get_or_else(Real.0)
    (-exponent) * (-base.log.get_or_else(Real.0)) = exponent * base.log.get_or_else(Real.0)
    exponent * base.log.get_or_else(Real.0) >= Real.0
    base.rpow(exponent) = Option.some((exponent * base.log.get_or_else(Real.0)).exp)
    exp_ge_one_of_nonneg(exponent * base.log.get_or_else(Real.0))
    (exponent * base.log.get_or_else(Real.0)).exp >= Real.1
    exists(value: Real) {
        value = (exponent * base.log.get_or_else(Real.0)).exp and
        base.rpow(exponent) = Option.some(value) and value >= Real.1
    }
}

/// For bases at least one, real powers are monotone in the exponent.
theorem rpow_exponent_monotone_base_ge_one(base: Real, a: Real, b: Real, va: Real, vb: Real) {
    base >= Real.1 and a <= b and base.rpow(a) = Option.some(va) and base.rpow(b) = Option.some(vb)
    implies va <= vb
} by {
    Real.1 > Real.0
    Real.1 <= base
    lt_of_lt_of_lte(Real.0, Real.1, base)
    Real.0 < base
    base > Real.0
    log_some_of_pos_exists(base)
    let lb: Real satisfy {
        base.log = Option.some(lb)
    }
    option_get_or_else_some[Real](lb, Real.0)
    option_get_or_else(Option.some(lb), Real.0) = lb
    base.log.get_or_else(Real.0) = lb
    base.log = Option.some(base.log.get_or_else(Real.0))
    log_nonneg_of_ge_one(base, base.log.get_or_else(Real.0))
    base.log.get_or_else(Real.0) >= Real.0
    not base.log.get_or_else(Real.0).is_negative
    lte_mul_nonneg_right(a, b, base.log.get_or_else(Real.0))
    a * base.log.get_or_else(Real.0) <= b * base.log.get_or_else(Real.0)
    exp_monotone(a * base.log.get_or_else(Real.0), b * base.log.get_or_else(Real.0))
    (a * base.log.get_or_else(Real.0)).exp <= (b * base.log.get_or_else(Real.0)).exp
    base.rpow(a) = Option.some((a * base.log.get_or_else(Real.0)).exp)
    base.rpow(b) = Option.some((b * base.log.get_or_else(Real.0)).exp)
    Option.some(va) = Option.some((a * base.log.get_or_else(Real.0)).exp)
    Option.some(vb) = Option.some((b * base.log.get_or_else(Real.0)).exp)
    some_injective[Real](va, (a * base.log.get_or_else(Real.0)).exp)
    some_injective[Real](vb, (b * base.log.get_or_else(Real.0)).exp)
    va = (a * base.log.get_or_else(Real.0)).exp
    vb = (b * base.log.get_or_else(Real.0)).exp
    va <= vb
}
