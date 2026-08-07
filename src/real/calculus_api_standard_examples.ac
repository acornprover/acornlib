/// Standard example uses of the real calculus derivative APIs.

from data.basic.function_algebra import pointwise_add, pointwise_mul
from data.basic.functions import identity_fn, function_eq_transport_predicate_rev
from real.real_base import Real
from real.continuity_affine import affine_real
from real.continuity_cube import cube_real
from real.continuity_square import square_real
from real.derivative_basic import has_derivative_at
from real.derivative_affine_named import affine_real_eq_pointwise_affine
from real.derivative_polynomial_chain import square_real_has_derivative_at,
    square_real_differentiable_at, cube_real_differentiable_at
from real.calculus_api import is_derivative_fn, differentiable_everywhere,
    derivative_fn_constant, derivative_fn_identity,
    derivative_fn_affine_identity, differentiable_everywhere_constant,
    differentiable_everywhere_identity, differentiable_everywhere_affine_identity
from real.calculus_quotient_api import nonvanishing_everywhere_constant,
    derivative_fn_reciprocal, derivative_fn_div_const,
    differentiable_everywhere_reciprocal, differentiable_everywhere_div_const
from real.derivative_quotient import pointwise_div_real, pointwise_reciprocal_real

/// The named affine function has the constant slope as its global derivative function.
theorem derivative_fn_affine_real(a: Real, b: Real) {
    is_derivative_fn(affine_real(a, b), constant[Real, Real](a))
} by {
    define derivative_pred(h: Real -> Real) -> Bool {
        is_derivative_fn(h, constant[Real, Real](a))
    }
    derivative_fn_affine_identity(a, b)
    derivative_pred(pointwise_add(pointwise_mul(constant[Real, Real](a), identity_fn[Real]), constant[Real, Real](b)))
    affine_real_eq_pointwise_affine(a, b)
    function_eq_transport_predicate_rev(
        derivative_pred,
        affine_real(a, b),
        pointwise_add(pointwise_mul(constant[Real, Real](a), identity_fn[Real]), constant[Real, Real](b))
    )
    is_derivative_fn(affine_real(a, b), constant[Real, Real](a))
}

/// The named affine function is differentiable everywhere.
theorem differentiable_everywhere_affine_real(a: Real, b: Real) {
    differentiable_everywhere(affine_real(a, b))
} by {
    define differentiable_pred(h: Real -> Real) -> Bool {
        differentiable_everywhere(h)
    }
    differentiable_everywhere_affine_identity(a, b)
    differentiable_pred(
        pointwise_add(pointwise_mul(constant[Real, Real](a), identity_fn[Real]), constant[Real, Real](b))
    )
    affine_real_eq_pointwise_affine(a, b)
    function_eq_transport_predicate_rev(
        differentiable_pred,
        affine_real(a, b),
        pointwise_add(pointwise_mul(constant[Real, Real](a), identity_fn[Real]), constant[Real, Real](b))
    )
    differentiable_everywhere(affine_real(a, b))
}

/// The named square function has the global derivative function `x ↦ x + x`.
theorem derivative_fn_square_real {
    is_derivative_fn(square_real, pointwise_add(identity_fn[Real], identity_fn[Real]))
} by {
    forall(x: Real) {
        square_real_has_derivative_at(x)
        identity_fn[Real](x) = x
        x * Real.1 = x
        pointwise_add(identity_fn[Real], identity_fn[Real], x) = identity_fn[Real](x) + identity_fn[Real](x)
        pointwise_add(identity_fn[Real], identity_fn[Real], x) = x + x
        x * Real.1 + x * Real.1 = x + x
        has_derivative_at(square_real, x, pointwise_add(identity_fn[Real], identity_fn[Real], x))
    }
}

/// The named square function is differentiable everywhere.
theorem differentiable_everywhere_square_real {
    differentiable_everywhere(square_real)
} by {
    forall(x: Real) {
        square_real_differentiable_at(x)
    }
}

/// The named cube function is differentiable everywhere.
theorem differentiable_everywhere_cube_real {
    differentiable_everywhere(cube_real)
} by {
    forall(x: Real) {
        cube_real_differentiable_at(x)
    }
}

/// The function `x ↦ x / c` has global derivative `x ↦ 1 / c`.
theorem derivative_fn_identity_div_const(c: Real) {
    is_derivative_fn(
        pointwise_div_real(identity_fn[Real], constant[Real, Real](c)),
        pointwise_div_real(constant[Real, Real](Real.1), constant[Real, Real](c))
    )
} by {
    derivative_fn_identity
    derivative_fn_div_const(identity_fn[Real], constant[Real, Real](Real.1), c)
    is_derivative_fn(
        pointwise_div_real(identity_fn[Real], constant[Real, Real](c)),
        pointwise_div_real(constant[Real, Real](Real.1), constant[Real, Real](c))
    )
}

/// The function `x ↦ x / c` is differentiable everywhere.
theorem differentiable_everywhere_identity_div_const(c: Real) {
    differentiable_everywhere(pointwise_div_real(identity_fn[Real], constant[Real, Real](c)))
} by {
    differentiable_everywhere_identity
    differentiable_everywhere_div_const(identity_fn[Real], c)
    differentiable_everywhere(pointwise_div_real(identity_fn[Real], constant[Real, Real](c)))
}

/// Dividing an affine function by a constant transports its global derivative.
theorem derivative_fn_affine_div_const(a: Real, b: Real, c: Real) {
    is_derivative_fn(
        pointwise_div_real(affine_real(a, b), constant[Real, Real](c)),
        pointwise_div_real(constant[Real, Real](a), constant[Real, Real](c))
    )
} by {
    derivative_fn_affine_real(a, b)
    derivative_fn_div_const(affine_real(a, b), constant[Real, Real](a), c)
    is_derivative_fn(
        pointwise_div_real(affine_real(a, b), constant[Real, Real](c)),
        pointwise_div_real(constant[Real, Real](a), constant[Real, Real](c))
    )
}

/// Dividing an affine function by a constant preserves differentiability everywhere.
theorem differentiable_everywhere_affine_div_const(a: Real, b: Real, c: Real) {
    differentiable_everywhere(pointwise_div_real(affine_real(a, b), constant[Real, Real](c)))
} by {
    differentiable_everywhere_affine_real(a, b)
    differentiable_everywhere_div_const(affine_real(a, b), c)
    differentiable_everywhere(pointwise_div_real(affine_real(a, b), constant[Real, Real](c)))
}

/// Dividing the square function by a constant transports its global derivative.
theorem derivative_fn_square_div_const(c: Real) {
    is_derivative_fn(
        pointwise_div_real(square_real, constant[Real, Real](c)),
        pointwise_div_real(pointwise_add(identity_fn[Real], identity_fn[Real]), constant[Real, Real](c))
    )
} by {
    derivative_fn_square_real
    derivative_fn_div_const(square_real, pointwise_add(identity_fn[Real], identity_fn[Real]), c)
    is_derivative_fn(
        pointwise_div_real(square_real, constant[Real, Real](c)),
        pointwise_div_real(pointwise_add(identity_fn[Real], identity_fn[Real]), constant[Real, Real](c))
    )
}

/// Dividing the square function by a constant preserves differentiability everywhere.
theorem differentiable_everywhere_square_div_const(c: Real) {
    differentiable_everywhere(pointwise_div_real(square_real, constant[Real, Real](c)))
} by {
    differentiable_everywhere_square_real
    differentiable_everywhere_div_const(square_real, c)
    differentiable_everywhere(pointwise_div_real(square_real, constant[Real, Real](c)))
}

/// A nonzero constant reciprocal is a standard global reciprocal example.
theorem derivative_fn_reciprocal_constant(c: Real) {
    c != Real.0 implies is_derivative_fn(
        pointwise_reciprocal_real(constant[Real, Real](c)),
        pointwise_mul(
            pointwise_div_real(
                constant[Real, Real](-Real.1),
                pointwise_mul(constant[Real, Real](c), constant[Real, Real](c))
            ),
            constant[Real, Real](Real.0)
        )
    )
} by {
    if c != Real.0 {
        derivative_fn_constant(c)
        nonvanishing_everywhere_constant(c)
        derivative_fn_reciprocal(constant[Real, Real](c), constant[Real, Real](Real.0))
        is_derivative_fn(
            pointwise_reciprocal_real(constant[Real, Real](c)),
            pointwise_mul(
                pointwise_div_real(
                constant[Real, Real](-Real.1),
                pointwise_mul(constant[Real, Real](c), constant[Real, Real](c))
            ),
                constant[Real, Real](Real.0)
            )
        )
    }
}

/// A nonzero constant reciprocal is differentiable everywhere.
theorem differentiable_everywhere_reciprocal_constant(c: Real) {
    c != Real.0 implies differentiable_everywhere(pointwise_reciprocal_real(constant[Real, Real](c)))
} by {
    if c != Real.0 {
        differentiable_everywhere_constant(c)
        nonvanishing_everywhere_constant(c)
        differentiable_everywhere_reciprocal(constant[Real, Real](c))
        differentiable_everywhere(pointwise_reciprocal_real(constant[Real, Real](c)))
    }
}
