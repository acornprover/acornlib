from data.basic.set import Set, double_inclusion, empty_set_is_always_subset, universal_set_compl_is_empty,
    empty_set_compl_is_universal
from real.real_field import Real
from real.topology import closure, closed_set_eq_closure, empty_set_is_closed,
    empty_set_is_open, interior, is_bounded_real_set, is_closed_set, is_open_set,
    subset_of_bounded_real_set_is_bounded, universal_set_is_open
from real.topology_boundary import boundary, boundary_eq_set_difference_interior_of_closed_set,
    boundary_is_closed, boundary_subset_closure, boundary_subset_of_closed_set, exterior,
    closure_eq_interior_union_boundary
from real.topology_bounded_closure import closure_of_bounded_real_set_is_bounded
from real.topology_compact import closed_bounded_real_set_is_compact,
    closed_subset_of_compact_real_set_is_compact, compact_real_set_is_bounded,
    compact_real_set_is_closed, empty_real_set_is_compact, is_compact_real_set
from real.topology_interior_algebra import open_set_eq_interior

/// The boundary of a bounded real set is bounded.
theorem boundary_of_bounded_real_set_is_bounded(s: Set[Real]) {
    is_bounded_real_set(s) implies is_bounded_real_set(boundary(s))
} by {
    if is_bounded_real_set(s) {
        closure_of_bounded_real_set_is_bounded(s)
        is_bounded_real_set(closure(s))
        boundary_subset_closure(s)
        boundary(s).subset(closure(s))
        subset_of_bounded_real_set_is_bounded(boundary(s), closure(s))
        is_bounded_real_set(boundary(s))
    }
}

/// The boundary of a compact real set is closed.
theorem boundary_of_compact_real_set_is_closed(s: Set[Real]) {
    is_compact_real_set(s) implies is_closed_set(boundary(s))
} by {
    if is_compact_real_set(s) {
        boundary_is_closed(s)
    }
}

/// The boundary of a compact real set is bounded.
theorem boundary_of_compact_real_set_is_bounded(s: Set[Real]) {
    is_compact_real_set(s) implies is_bounded_real_set(boundary(s))
} by {
    if is_compact_real_set(s) {
        compact_real_set_is_bounded(s)
        is_bounded_real_set(s)
        boundary_of_bounded_real_set_is_bounded(s)
        is_bounded_real_set(boundary(s))
    }
}

/// The boundary of a compact real set is compact.
theorem boundary_of_compact_real_set_is_compact(s: Set[Real]) {
    is_compact_real_set(s) implies is_compact_real_set(boundary(s))
} by {
    if is_compact_real_set(s) {
        compact_real_set_is_closed(s)
        is_closed_set(s)
        boundary_subset_of_closed_set(s)
        boundary(s).subset(s)
        boundary_is_closed(s)
        is_closed_set(boundary(s))
        closed_subset_of_compact_real_set_is_compact(boundary(s), s)
        is_compact_real_set(boundary(s))
    }
}

/// The boundary of a closed bounded real set is compact.
theorem boundary_of_closed_bounded_real_set_is_compact(s: Set[Real]) {
    is_closed_set(s) and is_bounded_real_set(s) implies is_compact_real_set(boundary(s))
} by {
    if is_closed_set(s) and is_bounded_real_set(s) {
        closed_bounded_real_set_is_compact(s)
        is_compact_real_set(s)
        boundary_of_compact_real_set_is_compact(s)
        is_compact_real_set(boundary(s))
    }
}

/// The boundary of a compact real set is contained in the set.
theorem boundary_of_compact_real_set_subset(s: Set[Real]) {
    is_compact_real_set(s) implies boundary(s).subset(s)
} by {
    if is_compact_real_set(s) {
        compact_real_set_is_closed(s)
        is_closed_set(s)
        boundary_subset_of_closed_set(s)
        boundary(s).subset(s)
    }
}

/// For a compact real set, the boundary is the set minus its interior.
theorem boundary_of_compact_real_set_eq_difference_interior(s: Set[Real]) {
    is_compact_real_set(s) implies boundary(s) = s.difference(interior(s))
} by {
    if is_compact_real_set(s) {
        compact_real_set_is_closed(s)
        is_closed_set(s)
        boundary_eq_set_difference_interior_of_closed_set(s)
        boundary(s) = s.difference(interior(s))
    }
}

/// The exterior of a closed real set is its complement.
theorem exterior_of_closed_set_eq_complement(s: Set[Real]) {
    is_closed_set(s) implies exterior(s) = s.c
} by {
    if is_closed_set(s) {
        from real.topology_complements import complement_of_closed_is_open
        complement_of_closed_is_open(s)
        is_open_set(s.c)
        open_set_eq_interior(s.c)
        s.c = interior(s.c)
        exterior(s) = interior(s.c)
        exterior(s) = s.c
    }
}

/// For an open real set, the closure is the union of the set and its boundary.
theorem closure_of_open_set_eq_set_union_boundary(s: Set[Real]) {
    is_open_set(s) implies closure(s) = s.union(boundary(s))
} by {
    if is_open_set(s) {
        open_set_eq_interior(s)
        s = interior(s)
        closure_eq_interior_union_boundary(s)
        closure(s) = interior(s).union(boundary(s))
        closure(s) = s.union(boundary(s))
    }
}

/// For a compact real set, the closure is the union of its interior and boundary.
theorem closure_of_compact_real_set_eq_interior_union_boundary(s: Set[Real]) {
    is_compact_real_set(s) implies closure(s) = interior(s).union(boundary(s))
} by {
    if is_compact_real_set(s) {
        closure_eq_interior_union_boundary(s)
    }
}

/// The boundary of the empty real set is empty.
theorem boundary_empty_real_set {
    boundary(Set[Real].empty_set) = Set[Real].empty_set
} by {
    empty_set_is_closed
    is_closed_set(Set[Real].empty_set)
    closed_set_eq_closure(Set[Real].empty_set)
    Set[Real].empty_set = closure(Set[Real].empty_set)
    closure(Set[Real].empty_set) = Set[Real].empty_set
    boundary_subset_closure(Set[Real].empty_set)
    boundary(Set[Real].empty_set).subset(closure(Set[Real].empty_set))
    boundary(Set[Real].empty_set).subset(Set[Real].empty_set)
    empty_set_is_always_subset[Real](boundary(Set[Real].empty_set))
    Set[Real].empty_set.subset(boundary(Set[Real].empty_set))
    double_inclusion(boundary(Set[Real].empty_set), Set[Real].empty_set)
}

/// The boundary of the universal real set is empty.
theorem boundary_universal_real_set {
    boundary(Set[Real].universal_set) = Set[Real].empty_set
} by {
    universal_set_compl_is_empty[Real]
    Set[Real].universal_set.c = Set[Real].empty_set
    boundary_subset_closure(Set[Real].universal_set.c)
    boundary(Set[Real].universal_set.c).subset(closure(Set[Real].universal_set.c))
    boundary_empty_real_set
    boundary(Set[Real].empty_set) = Set[Real].empty_set
    boundary(Set[Real].universal_set.c).subset(closure(Set[Real].empty_set))
    empty_set_is_closed
    closed_set_eq_closure(Set[Real].empty_set)
    closure(Set[Real].empty_set) = Set[Real].empty_set
    boundary(Set[Real].universal_set.c).subset(Set[Real].empty_set)
    from real.topology_boundary import boundary_complement_eq
    boundary_complement_eq(Set[Real].universal_set)
    boundary(Set[Real].universal_set.c) = boundary(Set[Real].universal_set)
    boundary(Set[Real].universal_set).subset(Set[Real].empty_set)
    empty_set_is_always_subset[Real](boundary(Set[Real].universal_set))
    Set[Real].empty_set.subset(boundary(Set[Real].universal_set))
    double_inclusion(boundary(Set[Real].universal_set), Set[Real].empty_set)
}

/// The exterior of the empty real set is the universal real set.
theorem exterior_empty_real_set {
    exterior(Set[Real].empty_set) = Set[Real].universal_set
} by {
    empty_set_compl_is_universal[Real]
    Set[Real].empty_set.c = Set[Real].universal_set
    universal_set_is_open
    is_open_set(Set[Real].universal_set)
    open_set_eq_interior(Set[Real].universal_set)
    Set[Real].universal_set = interior(Set[Real].universal_set)
    exterior(Set[Real].empty_set) = interior(Set[Real].empty_set.c)
    exterior(Set[Real].empty_set) = interior(Set[Real].universal_set)
    exterior(Set[Real].empty_set) = Set[Real].universal_set
}

/// The exterior of the universal real set is empty.
theorem exterior_universal_real_set {
    exterior(Set[Real].universal_set) = Set[Real].empty_set
} by {
    universal_set_compl_is_empty[Real]
    Set[Real].universal_set.c = Set[Real].empty_set
    empty_set_is_open
    is_open_set(Set[Real].empty_set)
    open_set_eq_interior(Set[Real].empty_set)
    Set[Real].empty_set = interior(Set[Real].empty_set)
    exterior(Set[Real].universal_set) = interior(Set[Real].universal_set.c)
    exterior(Set[Real].universal_set) = interior(Set[Real].empty_set)
    exterior(Set[Real].universal_set) = Set[Real].empty_set
}

/// The boundary of the empty real set is compact.
theorem boundary_empty_real_set_is_compact {
    is_compact_real_set(boundary(Set[Real].empty_set))
} by {
    boundary_empty_real_set
    empty_real_set_is_compact
    is_compact_real_set(Set[Real].empty_set)
    is_compact_real_set(boundary(Set[Real].empty_set))
}

/// The boundary of the universal real set is compact.
theorem boundary_universal_real_set_is_compact {
    is_compact_real_set(boundary(Set[Real].universal_set))
} by {
    boundary_universal_real_set
    empty_real_set_is_compact
    is_compact_real_set(Set[Real].empty_set)
    is_compact_real_set(boundary(Set[Real].universal_set))
}
