/// Eventual equality consumers for bounded and vanishing real sequences.
from nat import Nat, lte_trans
from order import not_lte_imp_gt, lt_imp_lte
from data.basic.functions import compose
from real.real_base import Real, lt_trans
from real.real_series import add_seq, mul_seq, neg_seq
from real.prod_seq import prod_seq
from real.real_seq import converges, converges_to, limit, tail_bound,
    tail_bound_implies_is_close, converges_imp_converges_to
from real.limits import vanishes, tends_to_infinity, is_subsequence_index,
    subsequence, converges_compose_tends_to_infinity, add_subsequence_index,
    subsequence_index_tends_to_infinity, subsequence_eq_compose
from real.bounded_seq import is_bounded_seq, converges_imp_bounded_seq,
    finite_seq_real_abs_bounded, real_strict_upper_bound_pair
from real.asymptotic_bounds import bounded_seq_positive_bound,
    bounded_add_seq, bounded_neg_seq, bounded_mul_seq, bounded_prod_seq,
    vanishes_add_seq, vanishes_neg_seq, vanishes_mul_seq, bounded_mul_vanishing_seq,
    vanishing_mul_bounded_seq, tail_bounds_imp_converges_to,
    converges_to_zero_imp_vanishes
from real.sequence_eventual_bridge import seq_eventually_equal_real,
    real_sequence_eq_predicate
from data.nat.nat_eventually import eventually_after_nat, eventually_after_nat_at,
    eventually_predicate_nat_has_witness
from real.sequence_eventual_tail_transport import seq_eventually_equal_real_shift_add,
    seq_eventually_equal_real_subsequence

/// Eventual equality transports boundedness of real sequences.
theorem seq_eventually_equal_real_is_bounded_seq(a: Nat -> Real, b: Nat -> Real) {
    seq_eventually_equal_real(a, b) and is_bounded_seq(a) implies is_bounded_seq(b)
} by {
    if seq_eventually_equal_real(a, b) and is_bounded_seq(a) {
        bounded_seq_positive_bound(a)
        let bound: Real satisfy {
            bound.is_positive and forall(n: Nat) {
                a(n).abs < bound
            }
        }
        eventually_predicate_nat_has_witness(real_sequence_eq_predicate(a, b))
        let n0: Nat satisfy {
            eventually_after_nat(real_sequence_eq_predicate(a, b), n0)
        }
        finite_seq_real_abs_bounded(b, n0)
        let prefix_bound: Real satisfy {
            forall(i: Nat) {
                i <= n0 implies b(i).abs < prefix_bound
            }
        }
        real_strict_upper_bound_pair(prefix_bound, bound)
        let total_bound: Real satisfy {
            prefix_bound < total_bound and bound < total_bound
        }
        forall(n: Nat) {
            if n0 <= n {
                eventually_after_nat_at(real_sequence_eq_predicate(a, b), n0, n)
                real_sequence_eq_predicate(a, b, n)
                a(n) = b(n)
                b(n).abs = a(n).abs
                a(n).abs < bound
                b(n).abs < bound
                bound < total_bound
                lt_trans(b(n).abs, bound, total_bound)
                b(n).abs < total_bound
            } else {
                not n0 <= n
                not_lte_imp_gt[Nat](n0, n)
                n < n0
                lt_imp_lte[Nat](n, n0)
                n <= n0
                b(n).abs < prefix_bound
                prefix_bound < total_bound
                lt_trans(b(n).abs, prefix_bound, total_bound)
                b(n).abs < total_bound
            }
        }
        is_bounded_seq(b)
    }
}

/// Eventual equality transports zero-tail bounds of real sequences.
theorem seq_eventually_equal_real_zero_tail_bounds(a: Nat -> Real, b: Nat -> Real) {
    seq_eventually_equal_real(a, b) and vanishes(a) implies forall(eps: Real) {
        eps.is_positive implies exists(n: Nat) {
            tail_bound(b, Real.0, n, eps)
        }
    }
} by {
    if seq_eventually_equal_real(a, b) and vanishes(a) {
        vanishes(a)
        converges(a)
        converges_imp_converges_to(a)
        converges_to(a, limit(a))
        limit(a) = Real.0
        converges_to(a, Real.0)
        converges_to(a, Real.0) = forall(eps0: Real) {
            eps0.is_positive implies exists(n: Nat) {
                tail_bound(a, Real.0, n, eps0)
            }
        }
        eventually_predicate_nat_has_witness(real_sequence_eq_predicate(a, b))
        let ne: Nat satisfy {
            eventually_after_nat(real_sequence_eq_predicate(a, b), ne)
        }
        forall(eps: Real) {
            if eps.is_positive {
                eps.is_positive implies exists(n: Nat) {
                    tail_bound(a, Real.0, n, eps)
                }
                let nt: Nat satisfy {
                    tail_bound(a, Real.0, nt, eps)
                }
                let n0 = ne.max(nt)
                ne <= n0
                nt <= n0
                forall(n: Nat) {
                    if n0 <= n {
                        lte_trans(ne, n0, n)
                        ne <= n
                        lte_trans(nt, n0, n)
                        nt <= n
                        eventually_after_nat_at(real_sequence_eq_predicate(a, b), ne, n)
                        real_sequence_eq_predicate(a, b, n)
                        a(n) = b(n)
                        tail_bound_implies_is_close(a, Real.0, nt, eps, n)
                        a(n).is_close(Real.0, eps)
                        b(n).is_close(Real.0, eps)
                    }
                }
                tail_bound(b, Real.0, n0, eps)
                exists(n: Nat) {
                    tail_bound(b, Real.0, n, eps)
                }
            }
        }
    }
}

/// Eventual equality transports vanishing of real sequences.
theorem seq_eventually_equal_real_vanishes(a: Nat -> Real, b: Nat -> Real) {
    seq_eventually_equal_real(a, b) and vanishes(a) implies vanishes(b)
} by {
    if seq_eventually_equal_real(a, b) and vanishes(a) {
        seq_eventually_equal_real_zero_tail_bounds(a, b)
        tail_bounds_imp_converges_to(b, Real.0)
        converges_to_zero_imp_vanishes(b)
        vanishes(b)
    }
}

/// Boundedness is preserved by reindexing along a map tending to infinity.
theorem bounded_compose_tends_to_infinity(a: Nat -> Real, f: Nat -> Nat) {
    is_bounded_seq(a) and tends_to_infinity(f) implies is_bounded_seq(compose(a, f))
} by {
    if is_bounded_seq(a) and tends_to_infinity(f) {
        let bound: Real satisfy {
            forall(n: Nat) {
                a(n).abs < bound
            }
        }
        forall(n: Nat) {
            compose(a, f)(n) = a(f(n))
            compose(a, f)(n).abs = a(f(n)).abs
            a(f(n)).abs < bound
            compose(a, f)(n).abs < bound
        }
        is_bounded_seq(compose(a, f))
    }
}

/// Boundedness is preserved by deleting a finite prefix.
theorem bounded_shift_add(a: Nat -> Real, k: Nat) {
    is_bounded_seq(a) implies is_bounded_seq(compose(a, k.add))
} by {
    if is_bounded_seq(a) {
        add_subsequence_index(k)
        subsequence_index_tends_to_infinity(k.add)
        tends_to_infinity(k.add)
        bounded_compose_tends_to_infinity(a, k.add)
        is_bounded_seq(compose(a, k.add))
    }
}

/// Boundedness is preserved by selected subsequences.
theorem bounded_subsequence(a: Nat -> Real, f: Nat -> Nat) {
    is_bounded_seq(a) and is_subsequence_index(f) implies is_bounded_seq(subsequence(a, f))
} by {
    if is_bounded_seq(a) and is_subsequence_index(f) {
        subsequence_index_tends_to_infinity(f)
        tends_to_infinity(f)
        bounded_compose_tends_to_infinity(a, f)
        is_bounded_seq(compose(a, f))
        subsequence_eq_compose(a, f)
        is_bounded_seq(subsequence(a, f))
    }
}

/// Vanishing is preserved by reindexing along a map tending to infinity.
theorem vanishes_compose_tends_to_infinity(a: Nat -> Real, f: Nat -> Nat) {
    vanishes(a) and tends_to_infinity(f) implies vanishes(compose(a, f))
} by {
    if vanishes(a) and tends_to_infinity(f) {
        converges(a)
        limit(a) = Real.0
        converges_compose_tends_to_infinity(a, f)
        converges_to(compose(a, f), limit(a))
        converges_to(compose(a, f), Real.0)
        converges_to_zero_imp_vanishes(compose(a, f))
        vanishes(compose(a, f))
    }
}

/// Vanishing is preserved by deleting a finite prefix.
theorem vanishes_shift_add(a: Nat -> Real, k: Nat) {
    vanishes(a) implies vanishes(compose(a, k.add))
} by {
    if vanishes(a) {
        add_subsequence_index(k)
        subsequence_index_tends_to_infinity(k.add)
        tends_to_infinity(k.add)
        vanishes_compose_tends_to_infinity(a, k.add)
        vanishes(compose(a, k.add))
    }
}

/// Vanishing is preserved by selected subsequences.
theorem vanishes_subsequence(a: Nat -> Real, f: Nat -> Nat) {
    vanishes(a) and is_subsequence_index(f) implies vanishes(subsequence(a, f))
} by {
    if vanishes(a) and is_subsequence_index(f) {
        subsequence_index_tends_to_infinity(f)
        tends_to_infinity(f)
        vanishes_compose_tends_to_infinity(a, f)
        vanishes(compose(a, f))
        subsequence_eq_compose(a, f)
        vanishes(subsequence(a, f))
    }
}

/// Boundedness of a sum is invariant under eventual equality of both summands.
theorem bounded_add_seq_of_eventual_equal(a: Nat -> Real, b: Nat -> Real, c: Nat -> Real, d: Nat -> Real) {
    seq_eventually_equal_real(a, c) and seq_eventually_equal_real(b, d)
    and is_bounded_seq(a) and is_bounded_seq(b) implies is_bounded_seq(add_seq(c, d))
} by {
    if seq_eventually_equal_real(a, c) and seq_eventually_equal_real(b, d)
    and is_bounded_seq(a) and is_bounded_seq(b) {
        seq_eventually_equal_real_is_bounded_seq(a, c)
        seq_eventually_equal_real_is_bounded_seq(b, d)
        bounded_add_seq(c, d)
        is_bounded_seq(add_seq(c, d))
    }
}

/// Boundedness of a negated sequence is invariant under eventual equality.
theorem bounded_neg_seq_of_eventual_equal(a: Nat -> Real, b: Nat -> Real) {
    seq_eventually_equal_real(a, b) and is_bounded_seq(a) implies is_bounded_seq(neg_seq(b))
} by {
    if seq_eventually_equal_real(a, b) and is_bounded_seq(a) {
        seq_eventually_equal_real_is_bounded_seq(a, b)
        bounded_neg_seq(b)
        is_bounded_seq(neg_seq(b))
    }
}

/// Boundedness of a scalar multiple is invariant under eventual equality.
theorem bounded_mul_seq_of_eventual_equal(c: Real, a: Nat -> Real, b: Nat -> Real) {
    seq_eventually_equal_real(a, b) and is_bounded_seq(a) implies is_bounded_seq(mul_seq(c, b))
} by {
    if seq_eventually_equal_real(a, b) and is_bounded_seq(a) {
        seq_eventually_equal_real_is_bounded_seq(a, b)
        bounded_mul_seq(c, b)
        is_bounded_seq(mul_seq(c, b))
    }
}

/// Boundedness of a product is invariant under eventual equality of both factors.
theorem bounded_prod_seq_of_eventual_equal(a: Nat -> Real, b: Nat -> Real, c: Nat -> Real, d: Nat -> Real) {
    seq_eventually_equal_real(a, c) and seq_eventually_equal_real(b, d)
    and is_bounded_seq(a) and is_bounded_seq(b) implies is_bounded_seq(prod_seq(c, d))
} by {
    if seq_eventually_equal_real(a, c) and seq_eventually_equal_real(b, d)
    and is_bounded_seq(a) and is_bounded_seq(b) {
        seq_eventually_equal_real_is_bounded_seq(a, c)
        seq_eventually_equal_real_is_bounded_seq(b, d)
        bounded_prod_seq(c, d)
        is_bounded_seq(prod_seq(c, d))
    }
}

/// Vanishing of a sum is invariant under eventual equality of both summands.
theorem vanishes_add_seq_of_eventual_equal(a: Nat -> Real, b: Nat -> Real, c: Nat -> Real, d: Nat -> Real) {
    seq_eventually_equal_real(a, c) and seq_eventually_equal_real(b, d)
    and vanishes(a) and vanishes(b) implies vanishes(add_seq(c, d))
} by {
    if seq_eventually_equal_real(a, c) and seq_eventually_equal_real(b, d)
    and vanishes(a) and vanishes(b) {
        seq_eventually_equal_real_vanishes(a, c)
        seq_eventually_equal_real_vanishes(b, d)
        vanishes_add_seq(c, d)
        vanishes(add_seq(c, d))
    }
}

/// Vanishing of a negated sequence is invariant under eventual equality.
theorem vanishes_neg_seq_of_eventual_equal(a: Nat -> Real, b: Nat -> Real) {
    seq_eventually_equal_real(a, b) and vanishes(a) implies vanishes(neg_seq(b))
} by {
    if seq_eventually_equal_real(a, b) and vanishes(a) {
        seq_eventually_equal_real_vanishes(a, b)
        vanishes_neg_seq(b)
        vanishes(neg_seq(b))
    }
}

/// Vanishing of a scalar multiple is invariant under eventual equality.
theorem vanishes_mul_seq_of_eventual_equal(c: Real, a: Nat -> Real, b: Nat -> Real) {
    seq_eventually_equal_real(a, b) and vanishes(a) implies vanishes(mul_seq(c, b))
} by {
    if seq_eventually_equal_real(a, b) and vanishes(a) {
        seq_eventually_equal_real_vanishes(a, b)
        vanishes_mul_seq(c, b)
        vanishes(mul_seq(c, b))
    }
}

/// Multiplication by an eventually equal bounded factor preserves vanishing.
theorem vanishes_prod_seq_of_eventual_equal(a: Nat -> Real, b: Nat -> Real, c: Nat -> Real, d: Nat -> Real) {
    seq_eventually_equal_real(a, c) and seq_eventually_equal_real(b, d)
    and is_bounded_seq(a) and vanishes(b) implies vanishes(prod_seq(c, d))
} by {
    if seq_eventually_equal_real(a, c) and seq_eventually_equal_real(b, d)
    and is_bounded_seq(a) and vanishes(b) {
        seq_eventually_equal_real_is_bounded_seq(a, c)
        seq_eventually_equal_real_vanishes(b, d)
        bounded_mul_vanishing_seq(c, d)
        vanishes(prod_seq(c, d))
    }
}
