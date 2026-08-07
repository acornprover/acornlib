from list import map, partial, sum
from data.list.list_range_sum_bridge import partial_add_offset_sum,
    partial_add_offset_sum_comm, partial_interval_sum, partial_interval_sum_comm,
    partial_interval_suc, partial_range_take_drop_sum,
    sum_map_drop_range_length, sum_map_take_range_length
from data.list.list_prefix_suffix import list_take
from nat import Nat
from real.real_base import Real
from real.real_series import diff_partial, partial_tail_sub, tail

numerals Nat

/// A real partial sum splits at any earlier index into the previous partial sum
/// and the finite interval sum over the remaining indices.
theorem real_partial_interval_sum(f: Nat -> Real, a: Nat, b: Nat) {
    a <= b implies partial(f, b) = partial(f, a) + sum(map(a.until(b), f))
} by {
    partial_interval_sum[Real](f, a, b)
}

/// The same real partial-sum split with the interval block first.
theorem real_partial_interval_sum_comm(f: Nat -> Real, a: Nat, b: Nat) {
    a <= b implies sum(map(a.until(b), f)) + partial(f, a) = partial(f, b)
} by {
    partial_interval_sum_comm[Real](f, a, b)
}

/// For real-valued sequences, an interval sum is the difference of the two
/// corresponding partial sums.
theorem real_interval_sum_eq_partial_sub(f: Nat -> Real, a: Nat, b: Nat) {
    a <= b implies sum(map(a.until(b), f)) = partial(f, b) - partial(f, a)
} by {
    if a <= b {
        diff_partial(f, a, b)
        partial(f, b) - partial(f, a) = sum(map(a.until(b), f))
    }
}

/// A partial sum with an additive offset splits into the initial block and the
/// following offset-length interval.
theorem real_partial_add_offset_sum(f: Nat -> Real, a: Nat, k: Nat) {
    partial(f, a + k) = partial(f, a) + sum(map(a.until(a + k), f))
} by {
    partial_add_offset_sum[Real](f, a, k)
}

/// The same additive-offset split with the interval block first.
theorem real_partial_add_offset_sum_comm(f: Nat -> Real, a: Nat, k: Nat) {
    sum(map(a.until(a + k), f)) + partial(f, a) = partial(f, a + k)
} by {
    partial_add_offset_sum_comm[Real](f, a, k)
}

/// An offset interval sum is the difference between the later and earlier
/// partial sums.
theorem real_offset_interval_sum_eq_partial_sub(f: Nat -> Real, a: Nat, k: Nat) {
    sum(map(a.until(a + k), f)) = partial(f, a + k) - partial(f, a)
} by {
    a <= a + k
    real_interval_sum_eq_partial_sub(f, a, a + k)
}

/// The successor partial-sum step as an interval block.
theorem real_partial_interval_suc(f: Nat -> Real, n: Nat) {
    partial(f, n.suc) = partial(f, n) + sum(map(n.until(n.suc), f))
} by {
    partial_interval_suc[Real](f, n)
}

/// The partial sum of a tail is the corresponding offset interval sum.
theorem real_partial_tail_eq_interval_sum(f: Nat -> Real, a: Nat, k: Nat) {
    partial(tail(f, a), k) = sum(map(a.until(a + k), f))
} by {
    partial_tail_sub(f, a, k)
    partial(tail(f, a), k) = partial(f, a + k) - partial(f, a)
    real_offset_interval_sum_eq_partial_sub(f, a, k)
}

/// A tail partial sum plus the initial partial sum recovers the later partial
/// sum.
theorem real_partial_tail_add_initial(f: Nat -> Real, a: Nat, k: Nat) {
    partial(f, a) + partial(tail(f, a), k) = partial(f, a + k)
} by {
    real_partial_add_offset_sum(f, a, k)
    real_partial_tail_eq_interval_sum(f, a, k)
}

/// The same tail decomposition with the tail block first.
theorem real_partial_tail_add_initial_comm(f: Nat -> Real, a: Nat, k: Nat) {
    partial(tail(f, a), k) + partial(f, a) = partial(f, a + k)
} by {
    real_partial_tail_add_initial(f, a, k)
    partial(tail(f, a), k) + partial(f, a) = partial(f, a) + partial(tail(f, a), k)
}

/// A partial sum over a range splits into sums over the taken prefix and the
/// dropped suffix of the range list.
theorem real_partial_range_take_drop_sum(f: Nat -> Real, n: Nat, k: Nat) {
    partial(f, n) = sum(map(list_take(n.range, k), f)) + sum(map(n.range.drop(k), f))
} by {
    partial_range_take_drop_sum[Real](f, n, k)
}

/// Taking all entries of a range list gives the corresponding real partial sum.
theorem real_sum_map_take_range_length(f: Nat -> Real, n: Nat) {
    sum(map(list_take(n.range, n), f)) = partial(f, n)
} by {
    sum_map_take_range_length[Real](f, n)
}

/// Dropping all entries of a range list gives a zero real mapped sum.
theorem real_sum_map_drop_range_length(f: Nat -> Real, n: Nat) {
    sum(map(n.range.drop(n), f)) = Real.0
} by {
    sum_map_drop_range_length[Real](f, n)
}
