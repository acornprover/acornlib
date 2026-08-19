from nat import Nat, from_nat
from order import lte_antisymm, not_lt_imp_gte
from real.exp import Real, exp_add, exp_zero, exp_pos, mul_one_over
from real.exp_inequalities import exp_monotone
from real.exp_log_properties import exp_rpow_mul
from real.harmonic import real_one_div_pos
from real.log import log_one, log_some_of_pos_exists, log_exp, exp_neg, exp_injective, exp_log_or_zero, log_mul, log_rpow, rpow_nat
from real.log_inequalities import log_monotone

numerals Real

// =====================================================================
// Logarithm basic identities
// =====================================================================

/// The logarithm of one is zero (value form).
theorem log_value_one {
    (Real.1).log.get_or_else(Real.0) = Real.0
} by {
    log_one
    Real.1.log = Option.some(Real.0)
    option_get_or_else_some[Real](Real.0, Real.0)
    option_get_or_else(Option.some(Real.0), Real.0) = Real.0
    (Real.1).log.get_or_else(Real.0) = Real.0
}

/// The logarithm of Euler's number is one.
theorem log_e {
    Real.e.log = Option.some(Real.1)
} by {
    Real.e = (Real.1).exp
    log_exp(Real.1)
    ((Real.1).exp).log = Option.some(Real.1)
    Real.e.log = Option.some(Real.1)
}

/// The logarithm of Euler's number is one (value form).
theorem log_value_e {
    (Real.e).log.get_or_else(Real.0) = Real.1
} by {
    log_e
    Real.e.log = Option.some(Real.1)
    option_get_or_else_some[Real](Real.1, Real.0)
    option_get_or_else(Option.some(Real.1), Real.0) = Real.1
    (Real.e).log.get_or_else(Real.0) = Real.1
}

/// The logarithm recovers the argument of the exponential (value form).
theorem log_value_exp(x: Real) {
    (x.exp).log.get_or_else(Real.0) = x
} by {
    log_exp(x)
    (x.exp).log = Option.some(x)
    option_get_or_else_some[Real](x, Real.0)
    option_get_or_else(Option.some(x), Real.0) = x
    (x.exp).log.get_or_else(Real.0) = x
}

// =====================================================================
// Exponential basic identities
// =====================================================================

/// The exponential of a difference is the quotient of the exponentials.
theorem exp_sub(x: Real, y: Real) {
    (x - y).exp = x.exp / y.exp
} by {
    exp_add(x, -y)
    (x + -y).exp = x.exp * (-y).exp
    x + -y = x - y
    (x - y).exp = (x + -y).exp
    (x - y).exp = x.exp * (-y).exp
    exp_neg(y)
    (-y).exp = Real.1 / y.exp
    x.exp * (-y).exp = x.exp * (Real.1 / y.exp)
    exp_pos(y)
    y.exp > Real.0
    y.exp != Real.0
    mul_one_over(x, y.exp)
    x.exp * (Real.1 / y.exp) = x.exp / y.exp
    (x - y).exp = x.exp / y.exp
}

/// The exponential of an argument and its negation multiply to one.
theorem exp_mul_exp_neg(x: Real) {
    x.exp * (-x).exp = Real.1
} by {
    exp_add(x, -x)
    (x + -x).exp = x.exp * (-x).exp
    x + -x = Real.0
    (Real.0).exp = x.exp * (-x).exp
    exp_zero
    (Real.0).exp = Real.1
    x.exp * (-x).exp = Real.1
}

/// Real powers of an exponential are exponentials of the scaled argument (value form).
theorem exp_rpow_val(x: Real, r: Real, v: Real) {
    (x.exp).rpow(r) = Option.some(v) implies v = (x * r).exp
} by {
    if (x.exp).rpow(r) = Option.some(v) {
        exp_rpow_mul(x, r)
        (x.exp).rpow(r) = Option.some((x * r).exp)
        Option.some(v) = Option.some((x * r).exp)
        some_injective[Real](v, (x * r).exp)
        v = (x * r).exp
    }
}

/// The logarithm of a product is the sum of the logarithms (value form).
theorem log_value_mul(x: Real, y: Real) {
    x > Real.0 and y > Real.0 implies (x * y).log.get_or_else(Real.0) = x.log.get_or_else(Real.0) + y.log.get_or_else(Real.0)
} by {
    if x > Real.0 and y > Real.0 {
        log_some_of_pos_exists(x)
        let lx: Real satisfy {
            x.log = Option.some(lx)
        }
        log_some_of_pos_exists(y)
        let ly: Real satisfy {
            y.log = Option.some(ly)
        }
        log_mul(x, y, lx, ly)
        (x * y).log = Option.some(lx + ly)
        x.log.get_or_else(Real.0) = lx
        y.log.get_or_else(Real.0) = ly
        lx + ly = x.log.get_or_else(Real.0) + y.log.get_or_else(Real.0)
        (x * y).log = Option.some(x.log.get_or_else(Real.0) + y.log.get_or_else(Real.0))
        x.is_positive
        y.is_positive
        (x * y).is_positive
        x * y > Real.0
        log_some_of_pos_exists(x * y)
        let lxy: Real satisfy {
            (x * y).log = Option.some(lxy)
        }
        (x * y).log.get_or_else(Real.0) = lxy
        (x * y).log = Option.some((x * y).log.get_or_else(Real.0))
        Option.some((x * y).log.get_or_else(Real.0)) = Option.some(x.log.get_or_else(Real.0) + y.log.get_or_else(Real.0))
        some_injective[Real]((x * y).log.get_or_else(Real.0), x.log.get_or_else(Real.0) + y.log.get_or_else(Real.0))
        (x * y).log.get_or_else(Real.0) = x.log.get_or_else(Real.0) + y.log.get_or_else(Real.0)
    }
}

/// The logarithm of a product with explicit logarithm values.
theorem log_mul_val(x: Real, y: Real, lx: Real, ly: Real) {
    x > Real.0 and y > Real.0 and x.log = Option.some(lx) and y.log = Option.some(ly)
    implies (x * y).log = Option.some(lx + ly)
} by {
    if x > Real.0 and y > Real.0 and x.log = Option.some(lx) and y.log = Option.some(ly) {
        log_mul(x, y, lx, ly)
        (x * y).log = Option.some(lx + ly)
    }
}

/// The logarithm of a quotient is the difference of the logarithms (value form).
theorem log_value_div(x: Real, y: Real) {
    x > Real.0 and y > Real.0 implies (x / y).log.get_or_else(Real.0) = x.log.get_or_else(Real.0) - y.log.get_or_else(Real.0)
} by {
    if x > Real.0 and y > Real.0 {
        x / y > Real.0
        log_some_of_pos_exists(x / y)
        let lxy: Real satisfy {
            (x / y).log = Option.some(lxy)
        }
        exp_log_or_zero(x / y, lxy)
        lxy.exp = x / y
        (x / y).log.get_or_else(Real.0) = lxy
        ((x / y).log.get_or_else(Real.0)).exp = x / y
        log_some_of_pos_exists(x)
        let lx: Real satisfy {
            x.log = Option.some(lx)
        }
        exp_log_or_zero(x, lx)
        lx.exp = x
        x.log.get_or_else(Real.0) = lx
        (x.log.get_or_else(Real.0)).exp = x
        log_some_of_pos_exists(y)
        let ly: Real satisfy {
            y.log = Option.some(ly)
        }
        exp_log_or_zero(y, ly)
        ly.exp = y
        y.log.get_or_else(Real.0) = ly
        (y.log.get_or_else(Real.0)).exp = y
        exp_sub(x.log.get_or_else(Real.0), y.log.get_or_else(Real.0))
        (x.log.get_or_else(Real.0) - y.log.get_or_else(Real.0)).exp = (x.log.get_or_else(Real.0)).exp / (y.log.get_or_else(Real.0)).exp
        (x.log.get_or_else(Real.0)).exp / (y.log.get_or_else(Real.0)).exp = x / y
        (x.log.get_or_else(Real.0) - y.log.get_or_else(Real.0)).exp = x / y
        ((x / y).log.get_or_else(Real.0)).exp = (x.log.get_or_else(Real.0) - y.log.get_or_else(Real.0)).exp
        exp_injective((x / y).log.get_or_else(Real.0), x.log.get_or_else(Real.0) - y.log.get_or_else(Real.0))
        (x / y).log.get_or_else(Real.0) = x.log.get_or_else(Real.0) - y.log.get_or_else(Real.0)
    }
}

/// The logarithm of a quotient is the difference of the logarithms.
theorem log_div(x: Real, y: Real) {
    x > Real.0 and y > Real.0 implies (x / y).log = Option.some(x.log.get_or_else(Real.0) - y.log.get_or_else(Real.0))
} by {
    if x > Real.0 and y > Real.0 {
        log_value_div(x, y)
        (x / y).log.get_or_else(Real.0) = x.log.get_or_else(Real.0) - y.log.get_or_else(Real.0)
        x / y > Real.0
        log_some_of_pos_exists(x / y)
        let lz: Real satisfy {
            (x / y).log = Option.some(lz)
        }
        (x / y).log.get_or_else(Real.0) = lz
        (x / y).log = Option.some((x / y).log.get_or_else(Real.0))
        Option.some((x / y).log.get_or_else(Real.0)) = Option.some(x.log.get_or_else(Real.0) - y.log.get_or_else(Real.0))
        (x / y).log = Option.some(x.log.get_or_else(Real.0) - y.log.get_or_else(Real.0))
    }
}

/// The logarithm of a reciprocal is the negated logarithm (value form).
theorem log_value_recip(x: Real) {
    x > Real.0 implies (Real.1 / x).log.get_or_else(Real.0) = -x.log.get_or_else(Real.0)
} by {
    if x > Real.0 {
        log_value_div(Real.1, x)
        (Real.1 / x).log.get_or_else(Real.0) = (Real.1).log.get_or_else(Real.0) - x.log.get_or_else(Real.0)
        log_value_one
        (Real.1).log.get_or_else(Real.0) = Real.0
        (Real.1).log.get_or_else(Real.0) - x.log.get_or_else(Real.0) = -x.log.get_or_else(Real.0)
        (Real.1 / x).log.get_or_else(Real.0) = -x.log.get_or_else(Real.0)
    }
}

/// The logarithm of a reciprocal is the negated logarithm.
theorem log_recip(x: Real) {
    x > Real.0 implies (Real.1 / x).log = Option.some(-x.log.get_or_else(Real.0))
} by {
    if x > Real.0 {
        log_value_recip(x)
        (Real.1 / x).log.get_or_else(Real.0) = -x.log.get_or_else(Real.0)
        real_one_div_pos(x)
        Real.1 / x > Real.0
        log_some_of_pos_exists(Real.1 / x)
        let lz: Real satisfy {
            (Real.1 / x).log = Option.some(lz)
        }
        (Real.1 / x).log.get_or_else(Real.0) = lz
        (Real.1 / x).log = Option.some((Real.1 / x).log.get_or_else(Real.0))
        Option.some((Real.1 / x).log.get_or_else(Real.0)) = Option.some(-x.log.get_or_else(Real.0))
        (Real.1 / x).log = Option.some(-x.log.get_or_else(Real.0))
    }
}

/// The logarithm of a real power is the exponent times the logarithm of the base.
theorem log_rpow_val(base: Real, r: Real, v: Real) {
    base > Real.0 and base.rpow(r) = Option.some(v)
    implies v.log = Option.some(r * base.log.get_or_else(Real.0))
} by {
    if base > Real.0 and base.rpow(r) = Option.some(v) {
        base.rpow(r) = Option.some((r * base.log.get_or_else(Real.0)).exp)
        Option.some(v) = Option.some((r * base.log.get_or_else(Real.0)).exp)
        some_injective[Real](v, (r * base.log.get_or_else(Real.0)).exp)
        v = (r * base.log.get_or_else(Real.0)).exp
        log_rpow(base, r)
        ((r * base.log.get_or_else(Real.0)).exp).log = Option.some(r * base.log.get_or_else(Real.0))
        v.log = Option.some(r * base.log.get_or_else(Real.0))
    }
}

/// The logarithm of a real power equals the exponent times the base logarithm (value form).
theorem log_value_rpow(base: Real, r: Real, v: Real) {
    base > Real.0 and base.rpow(r) = Option.some(v)
    implies v.log.get_or_else(Real.0) = r * base.log.get_or_else(Real.0)
} by {
    if base > Real.0 and base.rpow(r) = Option.some(v) {
        log_rpow_val(base, r, v)
        v.log = Option.some(r * base.log.get_or_else(Real.0))
        base.rpow(r) = Option.some((r * base.log.get_or_else(Real.0)).exp)
        Option.some(v) = Option.some((r * base.log.get_or_else(Real.0)).exp)
        some_injective[Real](v, (r * base.log.get_or_else(Real.0)).exp)
        v = (r * base.log.get_or_else(Real.0)).exp
        exp_pos(r * base.log.get_or_else(Real.0))
        (r * base.log.get_or_else(Real.0)).exp > Real.0
        v > Real.0
        log_some_of_pos_exists(v)
        let lv: Real satisfy {
            v.log = Option.some(lv)
        }
        v.log.get_or_else(Real.0) = lv
        v.log = Option.some(v.log.get_or_else(Real.0))
        Option.some(v.log.get_or_else(Real.0)) = Option.some(r * base.log.get_or_else(Real.0))
        some_injective[Real](v.log.get_or_else(Real.0), r * base.log.get_or_else(Real.0))
        v.log.get_or_else(Real.0) = r * base.log.get_or_else(Real.0)
    }
}

/// The logarithm of a natural power is the natural multiple of the logarithm.
theorem log_nat_pow(x: Real, n: Nat) {
    x > Real.0 implies (x.pow(n)).log = Option.some(from_nat[Real](n) * x.log.get_or_else(Real.0))
} by {
    if x > Real.0 {
        rpow_nat(x, n)
        x.rpow(from_nat[Real](n)) = Option.some(x.pow(n))
        log_rpow_val(x, from_nat[Real](n), x.pow(n))
        (x.pow(n)).log = Option.some(from_nat[Real](n) * x.log.get_or_else(Real.0))
    }
}

// =====================================================================
// Logarithm monotonicity
// =====================================================================

/// The logarithm is monotone on positive reals (value form).
theorem log_value_monotone(x: Real, y: Real) {
    x > Real.0 and y > Real.0 and x <= y implies x.log.get_or_else(Real.0) <= y.log.get_or_else(Real.0)
} by {
    if x > Real.0 and y > Real.0 and x <= y {
        log_some_of_pos_exists(x)
        let lx: Real satisfy {
            x.log = Option.some(lx)
        }
        x.log.get_or_else(Real.0) = lx
        x.log = Option.some(x.log.get_or_else(Real.0))
        log_some_of_pos_exists(y)
        let ly: Real satisfy {
            y.log = Option.some(ly)
        }
        y.log.get_or_else(Real.0) = ly
        y.log = Option.some(y.log.get_or_else(Real.0))
        log_monotone(x, y, x.log.get_or_else(Real.0), y.log.get_or_else(Real.0))
        x.log.get_or_else(Real.0) <= y.log.get_or_else(Real.0)
    }
}

/// Ordered logarithms imply ordered arguments.
theorem log_le_imp_le(x: Real, y: Real) {
    x > Real.0 and y > Real.0 and x.log.get_or_else(Real.0) <= y.log.get_or_else(Real.0) implies x <= y
} by {
    if x > Real.0 and y > Real.0 and x.log.get_or_else(Real.0) <= y.log.get_or_else(Real.0) {
        exp_monotone(x.log.get_or_else(Real.0), y.log.get_or_else(Real.0))
        (x.log.get_or_else(Real.0)).exp <= (y.log.get_or_else(Real.0)).exp
        log_some_of_pos_exists(x)
        let lx: Real satisfy {
            x.log = Option.some(lx)
        }
        exp_log_or_zero(x, lx)
        lx.exp = x
        x.log.get_or_else(Real.0) = lx
        (x.log.get_or_else(Real.0)).exp = x
        log_some_of_pos_exists(y)
        let ly: Real satisfy {
            y.log = Option.some(ly)
        }
        exp_log_or_zero(y, ly)
        ly.exp = y
        y.log.get_or_else(Real.0) = ly
        (y.log.get_or_else(Real.0)).exp = y
        x <= y
    }
}

/// Reverse-ordered logarithms imply reverse-ordered arguments.
theorem log_ge_imp_ge(x: Real, y: Real) {
    x > Real.0 and y > Real.0 and x.log.get_or_else(Real.0) >= y.log.get_or_else(Real.0) implies x >= y
} by {
    if x > Real.0 and y > Real.0 and x.log.get_or_else(Real.0) >= y.log.get_or_else(Real.0) {
        y.log.get_or_else(Real.0) <= x.log.get_or_else(Real.0)
        exp_monotone(y.log.get_or_else(Real.0), x.log.get_or_else(Real.0))
        (y.log.get_or_else(Real.0)).exp <= (x.log.get_or_else(Real.0)).exp
        log_some_of_pos_exists(y)
        let ly: Real satisfy {
            y.log = Option.some(ly)
        }
        exp_log_or_zero(y, ly)
        ly.exp = y
        y.log.get_or_else(Real.0) = ly
        (y.log.get_or_else(Real.0)).exp = y
        log_some_of_pos_exists(x)
        let lx: Real satisfy {
            x.log = Option.some(lx)
        }
        exp_log_or_zero(x, lx)
        lx.exp = x
        x.log.get_or_else(Real.0) = lx
        (x.log.get_or_else(Real.0)).exp = x
        y <= x
        x >= y
    }
}

/// Strictly ordered logarithms imply strictly ordered arguments.
theorem log_lt_imp_lt(x: Real, y: Real) {
    x > Real.0 and y > Real.0 and x.log.get_or_else(Real.0) < y.log.get_or_else(Real.0) implies x < y
} by {
    if x > Real.0 and y > Real.0 and x.log.get_or_else(Real.0) < y.log.get_or_else(Real.0) {
        x.log.get_or_else(Real.0) <= y.log.get_or_else(Real.0)
        log_le_imp_le(x, y)
        x <= y
        if not x < y {
            not_lt_imp_gte(x, y)
            x >= y
            lte_antisymm(x, y)
            x = y
            x.log.get_or_else(Real.0) = y.log.get_or_else(Real.0)
            false
        }
        x < y
    }
}

/// The logarithm is injective on positive reals (value form).
theorem log_injective(x: Real, y: Real) {
    x > Real.0 and y > Real.0 and x.log.get_or_else(Real.0) = y.log.get_or_else(Real.0) implies x = y
} by {
    if x > Real.0 and y > Real.0 and x.log.get_or_else(Real.0) = y.log.get_or_else(Real.0) {
        log_some_of_pos_exists(x)
        let lx: Real satisfy {
            x.log = Option.some(lx)
        }
        exp_log_or_zero(x, lx)
        lx.exp = x
        x.log.get_or_else(Real.0) = lx
        (x.log.get_or_else(Real.0)).exp = x
        log_some_of_pos_exists(y)
        let ly: Real satisfy {
            y.log = Option.some(ly)
        }
        exp_log_or_zero(y, ly)
        ly.exp = y
        y.log.get_or_else(Real.0) = ly
        (y.log.get_or_else(Real.0)).exp = y
        (x.log.get_or_else(Real.0)).exp = (y.log.get_or_else(Real.0)).exp
        x = y
    }
}

/// The logarithm is injective on positive reals (option form).
theorem log_injective_opt(x: Real, y: Real, lx: Real, ly: Real) {
    x > Real.0 and y > Real.0 and x.log = Option.some(lx) and y.log = Option.some(ly) and lx = ly
    implies x = y
} by {
    if x > Real.0 and y > Real.0 and x.log = Option.some(lx) and y.log = Option.some(ly) and lx = ly {
        x.log.get_or_else(Real.0) = lx
        y.log.get_or_else(Real.0) = ly
        x.log.get_or_else(Real.0) = y.log.get_or_else(Real.0)
        log_injective(x, y)
        x = y
    }
}
