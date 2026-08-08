from nat import Nat
from list import List, partial, sum, map
from data.basic.functions import flip
from algebra.add_semigroup import add_fn
from real.real_series import Real, is_lower_bound, seq_lte, is_increasing, is_upper_bound, prod_fn, increasing_is_monotone
from real.real_ring import mul_assoc, real_mul_comm
from real.real_seq import converges, limit
from order import is_monotone

numerals Real
numerals Nat

/// Convert a natural number to a real number.
define nat_to_real(n: Nat) -> Real {
    match n {
        Nat.zero {
            Real.0
        }
        Nat.suc(pred) {
            nat_to_real(pred) + Real.1
        }
    }
}

/// This file defines rectangular sums and proves theorems about them,
/// working towards Tonelli's theorem.

attributes Real {
}

/// True if f is nonnegative everywhere.
define nonneg_fn_2(f: (Nat, Nat) -> Real) -> Bool {
    forall(i: Nat, j: Nat) {
        f(i, j) >= Real.0
    }
}

/// True if bound is an upper bound for f everywhere.
define is_upper_bound_fn_2(f: (Nat, Nat) -> Real, bound: Real) -> Bool {
    forall(i: Nat, j: Nat) {
        f(i, j) <= bound
    }
}

/// True if bound is a lower bound for f everywhere.
define is_lower_bound_fn_2(f: (Nat, Nat) -> Real, bound: Real) -> Bool {
    forall(i: Nat, j: Nat) {
        f(i, j) >= bound
    }
}

/// True if f is pointwise less than or equal to g.
define lte_fn_2(f: (Nat, Nat) -> Real, g: (Nat, Nat) -> Real) -> Bool {
    forall(i: Nat, j: Nat) {
        f(i, j) <= g(i, j)
    }
}

/// Constant two-argument function.
define const_fn_2(c: Real, i: Nat, j: Nat) -> Real {
    c
}

/// Pointwise addition of two-argument functions.
define add_fn_2(f: (Nat, Nat) -> Real, g: (Nat, Nat) -> Real, i: Nat, j: Nat) -> Real {
    f(i, j) + g(i, j)
}

/// Pointwise subtraction of two-argument functions.
define sub_fn_2(f: (Nat, Nat) -> Real, g: (Nat, Nat) -> Real, i: Nat, j: Nat) -> Real {
    f(i, j) - g(i, j)
}

/// Scalar multiplication of a two-argument function.
define scalar_mul_fn_2(c: Real, f: (Nat, Nat) -> Real, i: Nat, j: Nat) -> Real {
    c * f(i, j)
}

/// Shift the first argument of f by offset.
define shift_rows(f: (Nat, Nat) -> Real, offset: Nat, i: Nat, j: Nat) -> Real {
    f(offset + i, j)
}

/// Shift the second argument of f by offset.
define shift_cols(f: (Nat, Nat) -> Real, offset: Nat, i: Nat, j: Nat) -> Real {
    f(i, offset + j)
}

/// Helper function to compute the sum over row i.
define row_sum(f: (Nat, Nat) -> Real, m: Nat, i: Nat) -> Real {
    partial(f(i), m)
}

/// Helper function to compute the sum over column j.
define col_sum(f: (Nat, Nat) -> Real, n: Nat, j: Nat) -> Real {
    partial(flip(f, j), n)
}

/// The sum of f over an m × n rectangle.
/// Computes sum_{i=0}^{m-1} sum_{j=0}^{n-1} f(i, j).
define rectangular_sum(f: (Nat, Nat) -> Real, m: Nat, n: Nat) -> Real {
    partial(row_sum(f, n), m)
}

/// Alternative definition: sum columns first.
/// Computes sum_{j=0}^{n-1} sum_{i=0}^{m-1} f(i, j).
define rectangular_sum_col_first(f: (Nat, Nat) -> Real, m: Nat, n: Nat) -> Real {
    partial(col_sum(f, m), n)
}

/// Rectangular sums with the column bound fixed.
define rectangular_sum_by_rows(f: (Nat, Nat) -> Real, n: Nat, m: Nat) -> Real {
    rectangular_sum(f, m, n)
}

/// Rectangular sums with the row bound fixed.
define rectangular_sum_by_cols(f: (Nat, Nat) -> Real, m: Nat, n: Nat) -> Real {
    rectangular_sum(f, m, n)
}


/// The sum over an empty rectangle (m=0 or n=0) is zero.
theorem rectangular_sum_zero_rows(f: (Nat, Nat) -> Real, n: Nat) {
    rectangular_sum(f, Nat.0, n) = Real.0
}

theorem rectangular_sum_zero_cols(f: (Nat, Nat) -> Real, m: Nat) {
    rectangular_sum(f, m, Nat.0) = Real.0
} by {
    define p(k: Nat) -> Bool {
        rectangular_sum(f, k, Nat.0) = Real.0
    }

    p(Nat.0)

    forall(k: Nat) {
        if p(k) {
            row_sum(f, Nat.0, k) = Real.0
            rectangular_sum(f, k, Nat.0) = Real.0
            partial[Real](row_sum(f, Nat.0), k) = rectangular_sum(f, k, Nat.0)
            partial[Real](row_sum(f, Nat.0), k) = Real.0
            partial[Real](row_sum(f, Nat.0), k) + row_sum(f, Nat.0, k) = partial[Real](row_sum(f, Nat.0), k.suc)
            Real.0 + Real.0 = partial[Real](row_sum(f, Nat.0), k.suc)
            partial[Real](row_sum(f, Nat.0), k.suc) = rectangular_sum(f, k.suc, Nat.0)
            rectangular_sum(f, k.suc, Nat.0) = Real.0
            p(k.suc)
        }
    }
    p(m)
}

/// Fubini's theorem for finite rectangular sums: we can exchange the order of summation.
theorem rectangular_fubini(f: (Nat, Nat) -> Real, m: Nat, n: Nat) {
    rectangular_sum(f, m, n) = rectangular_sum_col_first(f, m, n)
} by {
    // Proof by induction on m
    define p(k: Nat) -> Bool {
        forall(n2: Nat) {
            rectangular_sum(f, k, n2) = rectangular_sum_col_first(f, k, n2)
        }
    }

    // Base case: k = 0
    forall(n2: Nat) {
        rectangular_sum(f, Nat.0, n2) = Real.0

        // Prove rectangular_sum_col_first(f, 0, n2) = 0 by induction on n2
        define q(n3: Nat) -> Bool {
            rectangular_sum_col_first(f, Nat.0, n3) = Real.0
        }

        q(Nat.0)

        forall(n3: Nat) {
            if q(n3) {
                col_sum(f, Nat.0, n3) = Real.0
                rectangular_sum_col_first(f, Nat.0, n3) = Real.0
                partial[Real](col_sum(f, Nat.0), n3) = rectangular_sum_col_first(f, Nat.0, n3)
                partial[Real](col_sum(f, Nat.0), n3) = Real.0
                partial[Real](col_sum(f, Nat.0), n3) + col_sum(f, Nat.0, n3) = partial[Real](col_sum(f, Nat.0), n3.suc)
                Real.0 + Real.0 = partial[Real](col_sum(f, Nat.0), n3.suc)
                partial[Real](col_sum(f, Nat.0), n3.suc) = rectangular_sum_col_first(f, Nat.0, n3.suc)
                rectangular_sum_col_first(f, Nat.0, n3.suc) = Real.0
                q(n3.suc)
            }
        }

        q(n2)
        rectangular_sum_col_first(f, Nat.0, n2) = Real.0
    }
    p(Nat.0)

    // Inductive step
    forall(k: Nat) {
        if p(k) {
            forall(n2: Nat) {
                // By IH: rectangular_sum(f, k, n2) = rectangular_sum_col_first(f, k, n2)

                // First prove that col_sum(f, k.suc, j) = col_sum(f, k, j) + f(k, j)
                define helper(j: Nat) -> Bool {
                    col_sum(f, k.suc, j) = col_sum(f, k, j) + f(k, j)
                }

                forall(j: Nat) {
                    col_sum(f, k.suc, j) = partial(flip(f, j), k.suc)
                    partial(flip(f, j), k.suc) = partial(flip(f, j), k) + flip(f, j)(k)
                    flip(f, j)(k) = f(k, j)
                    col_sum(f, k, j) = partial(flip(f, j), k)
                    col_sum(f, k.suc, j) = col_sum(f, k, j) + f(k, j)
                    helper(j)
                }

                // Now prove that partial sums are equal
                define r(n3: Nat) -> Bool {
                    partial(col_sum(f, k.suc), n3) = partial(col_sum(f, k), n3) + partial(f(k), n3)
                }

                r(Nat.0)

                forall(n3: Nat) {
                    if r(n3) {
                        col_sum(f, k.suc, n3) = col_sum(f, k, n3) + f(k, n3)
                        partial(col_sum(f, k.suc), n3) = partial(col_sum(f, k), n3) + partial(f(k), n3)
                        partial(col_sum(f, k.suc), n3.suc) = partial(col_sum(f, k.suc), n3) + col_sum(f, k.suc, n3)
                        partial(col_sum(f, k.suc), n3.suc) = partial(col_sum(f, k.suc), n3) + (col_sum(f, k, n3) + f(k, n3))
                        partial(col_sum(f, k.suc), n3.suc) = (partial(col_sum(f, k), n3) + partial(f(k), n3)) + (col_sum(f, k, n3) + f(k, n3))
                        partial(col_sum(f, k.suc), n3.suc) = (partial(col_sum(f, k), n3) + col_sum(f, k, n3)) + (partial(f(k), n3) + f(k, n3))
                        partial(col_sum(f, k), n3.suc) = partial(col_sum(f, k), n3) + col_sum(f, k, n3)
                        partial(f(k), n3.suc) = partial(f(k), n3) + f(k, n3)
                        partial(col_sum(f, k.suc), n3.suc) = partial(col_sum(f, k), n3.suc) + partial(f(k), n3.suc)
                        r(n3.suc)
                    }
                }

                r(n2)
                partial(col_sum(f, k.suc), n2) = partial(col_sum(f, k), n2) + partial(f(k), n2)
                row_sum(f, n2, k) = partial(f(k), n2)
                partial(col_sum(f, k.suc), n2) = partial(col_sum(f, k), n2) + row_sum(f, n2, k)
                rectangular_sum_col_first(f, k.suc, n2) = rectangular_sum_col_first(f, k, n2) + row_sum(f, n2, k)

                rectangular_sum(f, k.suc, n2) = rectangular_sum(f, k, n2) + row_sum(f, n2, k)

                rectangular_sum(f, k.suc, n2) = rectangular_sum_col_first(f, k.suc, n2)
            }
            p(k.suc)
        }
    }
    p(m)
}

/// Rectangular sum is monotone for nonnegative functions.
theorem rectangular_sum_nonneg(f: (Nat, Nat) -> Real, m: Nat, n: Nat) {
    nonneg_fn_2(f)
    implies
    rectangular_sum(f, m, n) >= Real.0
} by {
    define p(k: Nat) -> Bool {
        forall(n2: Nat) {
            nonneg_fn_2(f)
            implies
            rectangular_sum(f, k, n2) >= Real.0
        }
    }

    p(Nat.0)

    forall(k: Nat) {
        if p(k) {
            forall(n2: Nat) {
                if nonneg_fn_2(f) {
                    // Prove row_sum(f, n2, k) >= 0
                    define r(n3: Nat) -> Bool {
                        row_sum(f, n3, k) >= Real.0
                    }

                    row_sum(f, Nat.0, k) = partial(f(k), Nat.0)
                    partial(f(k), Nat.0) = Real.0
                    row_sum(f, Nat.0, k) = Real.0
                    Real.0 >= Real.0
                    row_sum(f, Nat.0, k) >= Real.0
                    r(Nat.0)

                    forall(n3: Nat) {
                        if r(n3) {
                            f(k, n3) >= Real.0
                            Real.0 + Real.0 <= row_sum(f, n3, k) + f(k, n3)
                            Real.0 <= row_sum(f, n3, k) + f(k, n3)
                            row_sum(f, n3, k) + f(k, n3) >= Real.0
                            row_sum(f, n3.suc, k) = row_sum(f, n3, k) + f(k, n3)
                            row_sum(f, n3.suc, k) >= Real.0
                            r(n3.suc)
                        }
                    }

                    r(n2)
                    row_sum(f, n2, k) >= Real.0
                    rectangular_sum(f, k, n2) >= Real.0
                    Real.0 + Real.0 <= rectangular_sum(f, k, n2) + row_sum(f, n2, k)
                    Real.0 <= rectangular_sum(f, k, n2) + row_sum(f, n2, k)
                    rectangular_sum(f, k, n2) + row_sum(f, n2, k) >= Real.0
                    rectangular_sum(f, k.suc, n2) = rectangular_sum(f, k, n2) + row_sum(f, n2, k)
                    rectangular_sum(f, k.suc, n2) >= Real.0
                }
            }
            p(k.suc)
        }
    }
    p(m)
}

/// If f <= g pointwise, then their rectangular sums satisfy the same inequality.
theorem rectangular_sum_monotone(f: (Nat, Nat) -> Real, g: (Nat, Nat) -> Real, m: Nat, n: Nat) {
    lte_fn_2(f, g)
    implies
    rectangular_sum(f, m, n) <= rectangular_sum(g, m, n)
} by {
    define p(k: Nat) -> Bool {
        forall(n2: Nat) {
            lte_fn_2(f, g)
            implies
            rectangular_sum(f, k, n2) <= rectangular_sum(g, k, n2)
        }
    }

    p(Nat.0)

    forall(k: Nat) {
        if p(k) {
            forall(n2: Nat) {
                if lte_fn_2(f, g) {
                    // Prove row_sum(f, n2, k) <= row_sum(g, n2, k)
                    define r(n3: Nat) -> Bool {
                        row_sum(f, n3, k) <= row_sum(g, n3, k)
                    }

                    row_sum(f, Nat.0, k) = partial(f(k), Nat.0)
                    partial(f(k), Nat.0) = Real.0
                    row_sum(f, Nat.0, k) = Real.0
                    row_sum(g, Nat.0, k) = partial(g(k), Nat.0)
                    partial(g(k), Nat.0) = Real.0
                    row_sum(g, Nat.0, k) = Real.0
                    Real.0 <= Real.0
                    row_sum(f, Nat.0, k) <= row_sum(g, Nat.0, k)
                    r(Nat.0)

                    forall(n3: Nat) {
                        if r(n3) {
                            f(k, n3) <= g(k, n3)
                            row_sum(f, n3, k) <= row_sum(g, n3, k)
                            row_sum(f, n3, k) + f(k, n3) <= row_sum(g, n3, k) + g(k, n3)
                            row_sum(f, n3.suc, k) = row_sum(f, n3, k) + f(k, n3)
                            row_sum(g, n3.suc, k) = row_sum(g, n3, k) + g(k, n3)
                            row_sum(f, n3.suc, k) <= row_sum(g, n3.suc, k)
                            r(n3.suc)
                        }
                    }

                    r(n2)
                    row_sum(f, n2, k) <= row_sum(g, n2, k)
                    rectangular_sum(f, k, n2) <= rectangular_sum(g, k, n2)
                    rectangular_sum(f, k, n2) + row_sum(f, n2, k) <= rectangular_sum(g, k, n2) + row_sum(g, n2, k)
                    rectangular_sum(f, k.suc, n2) <= rectangular_sum(g, k.suc, n2)
                }
            }
            p(k.suc)
        }
    }
    p(m)
}

/// Rectangular sum is additive in the function.
theorem rectangular_sum_add(f: (Nat, Nat) -> Real, g: (Nat, Nat) -> Real, m: Nat, n: Nat) {
    rectangular_sum(add_fn_2(f, g), m, n)
    =
    rectangular_sum(f, m, n) + rectangular_sum(g, m, n)
} by {
    define p(k: Nat) -> Bool {
        rectangular_sum(add_fn_2(f, g), k, n)
        =
        rectangular_sum(f, k, n) + rectangular_sum(g, k, n)
    }
    rectangular_sum(add_fn_2(f, g), Nat.0, n) = Real.0
    rectangular_sum(f, Nat.0, n) = Real.0
    rectangular_sum(g, Nat.0, n) = Real.0
    Real.0 + Real.0 = Real.0
    rectangular_sum(f, Nat.0, n) + rectangular_sum(g, Nat.0, n) = Real.0
    rectangular_sum(add_fn_2(f, g), Nat.0, n) = rectangular_sum(f, Nat.0, n) + rectangular_sum(g, Nat.0, n)
    p(Nat.0)

    forall(k: Nat) {
        if p(k) {
            define r(n3: Nat) -> Bool {
                row_sum(add_fn_2(f, g), n3, k) = row_sum(f, n3, k) + row_sum(g, n3, k)
            }
            row_sum(add_fn_2(f, g), Nat.0, k) = partial(add_fn_2(f, g, k), Nat.0)
            partial(add_fn_2(f, g, k), Nat.0) = Real.0
            row_sum(add_fn_2(f, g), Nat.0, k) = Real.0
            row_sum(f, Nat.0, k) = partial(f(k), Nat.0)
            partial(f(k), Nat.0) = Real.0
            row_sum(f, Nat.0, k) = Real.0
            row_sum(g, Nat.0, k) = partial(g(k), Nat.0)
            partial(g(k), Nat.0) = Real.0
            row_sum(g, Nat.0, k) = Real.0
            Real.0 + Real.0 = Real.0
            row_sum(f, Nat.0, k) + row_sum(g, Nat.0, k) = Real.0
            row_sum(add_fn_2(f, g), Nat.0, k) = row_sum(f, Nat.0, k) + row_sum(g, Nat.0, k)
            r(Nat.0)

            forall(n3: Nat) {
                if r(n3) {
                    row_sum(add_fn_2(f, g), n3, k) = row_sum(f, n3, k) + row_sum(g, n3, k)
                    row_sum(add_fn_2(f, g), n3.suc, k) = row_sum(add_fn_2(f, g), n3, k) + add_fn_2(f, g, k, n3)
                    add_fn_2(f, g, k, n3) = f(k, n3) + g(k, n3)
                    row_sum(add_fn_2(f, g), n3.suc, k) = row_sum(add_fn_2(f, g), n3, k) + (f(k, n3) + g(k, n3))
                    row_sum(add_fn_2(f, g), n3.suc, k) = (row_sum(f, n3, k) + row_sum(g, n3, k)) + (f(k, n3) + g(k, n3))
                    (row_sum(f, n3, k) + row_sum(g, n3, k)) + (f(k, n3) + g(k, n3)) = row_sum(f, n3, k) + row_sum(g, n3, k) + f(k, n3) + g(k, n3)
                    row_sum(f, n3, k) + row_sum(g, n3, k) + f(k, n3) = row_sum(f, n3, k) + f(k, n3) + row_sum(g, n3, k)
                    row_sum(f, n3, k) + row_sum(g, n3, k) + f(k, n3) + g(k, n3) = row_sum(f, n3, k) + f(k, n3) + row_sum(g, n3, k) + g(k, n3)
                    row_sum(f, n3, k) + f(k, n3) + row_sum(g, n3, k) + g(k, n3) = (row_sum(f, n3, k) + f(k, n3)) + (row_sum(g, n3, k) + g(k, n3))
                    row_sum(add_fn_2(f, g), n3.suc, k) = (row_sum(f, n3, k) + f(k, n3)) + (row_sum(g, n3, k) + g(k, n3))
                    row_sum(f, n3.suc, k) = row_sum(f, n3, k) + f(k, n3)
                    row_sum(g, n3.suc, k) = row_sum(g, n3, k) + g(k, n3)
                    row_sum(add_fn_2(f, g), n3.suc, k) = row_sum(f, n3.suc, k) + row_sum(g, n3.suc, k)
                    r(n3.suc)
                }
            }

            r(n)
            rectangular_sum(add_fn_2(f, g), k.suc, n) = rectangular_sum(add_fn_2(f, g), k, n) + row_sum(add_fn_2(f, g), n, k)
            rectangular_sum(add_fn_2(f, g), k, n) = rectangular_sum(f, k, n) + rectangular_sum(g, k, n)
            row_sum(add_fn_2(f, g), n, k) = row_sum(f, n, k) + row_sum(g, n, k)
            rectangular_sum(add_fn_2(f, g), k.suc, n) = (rectangular_sum(f, k, n) + rectangular_sum(g, k, n)) + (row_sum(f, n, k) + row_sum(g, n, k))
            (rectangular_sum(f, k, n) + rectangular_sum(g, k, n)) + (row_sum(f, n, k) + row_sum(g, n, k)) = rectangular_sum(f, k, n) + rectangular_sum(g, k, n) + row_sum(f, n, k) + row_sum(g, n, k)
            rectangular_sum(f, k, n) + rectangular_sum(g, k, n) + row_sum(f, n, k) = rectangular_sum(f, k, n) + row_sum(f, n, k) + rectangular_sum(g, k, n)
            rectangular_sum(f, k, n) + rectangular_sum(g, k, n) + row_sum(f, n, k) + row_sum(g, n, k) = rectangular_sum(f, k, n) + row_sum(f, n, k) + rectangular_sum(g, k, n) + row_sum(g, n, k)
            rectangular_sum(f, k, n) + row_sum(f, n, k) + rectangular_sum(g, k, n) + row_sum(g, n, k) = (rectangular_sum(f, k, n) + row_sum(f, n, k)) + (rectangular_sum(g, k, n) + row_sum(g, n, k))
            rectangular_sum(add_fn_2(f, g), k.suc, n) = (rectangular_sum(f, k, n) + row_sum(f, n, k)) + (rectangular_sum(g, k, n) + row_sum(g, n, k))
            rectangular_sum(f, k.suc, n) = rectangular_sum(f, k, n) + row_sum(f, n, k)
            rectangular_sum(g, k.suc, n) = rectangular_sum(g, k, n) + row_sum(g, n, k)
            rectangular_sum(add_fn_2(f, g), k.suc, n) = rectangular_sum(f, k.suc, n) + rectangular_sum(g, k.suc, n)
            p(k.suc)
        }
    }

    p(m)
}

/// Scaling a rectangular sum.
theorem rectangular_sum_scale(c: Real, f: (Nat, Nat) -> Real, m: Nat, n: Nat) {
    rectangular_sum(scalar_mul_fn_2(c, f), m, n)
    =
    c * rectangular_sum(f, m, n)
} by {
    define p(k: Nat) -> Bool {
        forall(n2: Nat) {
            rectangular_sum(scalar_mul_fn_2(c, f), k, n2)
            =
            c * rectangular_sum(f, k, n2)
        }
    }

    p(Nat.0)

    forall(k: Nat) {
        if p(k) {
            forall(n2: Nat) {
                // Prove row_sum scales
                define r(n3: Nat) -> Bool {
                    row_sum(scalar_mul_fn_2(c, f), n3, k) = c * row_sum(f, n3, k)
                }

                row_sum(scalar_mul_fn_2(c, f), Nat.0, k) = partial(scalar_mul_fn_2(c, f, k), Nat.0)
                partial(scalar_mul_fn_2(c, f, k), Nat.0) = Real.0
                row_sum(scalar_mul_fn_2(c, f), Nat.0, k) = Real.0
                row_sum(f, Nat.0, k) = partial(f(k), Nat.0)
                partial(f(k), Nat.0) = Real.0
                row_sum(f, Nat.0, k) = Real.0
                c * Real.0 = Real.0
                c * row_sum(f, Nat.0, k) = Real.0
                row_sum(scalar_mul_fn_2(c, f), Nat.0, k) = c * row_sum(f, Nat.0, k)
                r(Nat.0)

                forall(n3: Nat) {
                    if r(n3) {
                        row_sum(scalar_mul_fn_2(c, f), n3, k) = c * row_sum(f, n3, k)
                        row_sum(scalar_mul_fn_2(c, f), n3.suc, k) = row_sum(scalar_mul_fn_2(c, f), n3, k) + scalar_mul_fn_2(c, f, k, n3)
                        scalar_mul_fn_2(c, f, k, n3) = c * f(k, n3)
                        row_sum(f, n3.suc, k) = row_sum(f, n3, k) + f(k, n3)
                        c * row_sum(f, n3, k) + c * f(k, n3) = c * (row_sum(f, n3, k) + f(k, n3))
                        row_sum(scalar_mul_fn_2(c, f), n3.suc, k) = c * row_sum(f, n3.suc, k)
                        r(n3.suc)
                    }
                }

                r(n2)
                row_sum(scalar_mul_fn_2(c, f), n2, k) = c * row_sum(f, n2, k)
                rectangular_sum(scalar_mul_fn_2(c, f), k.suc, n2) = rectangular_sum(scalar_mul_fn_2(c, f), k, n2) + row_sum(scalar_mul_fn_2(c, f), n2, k)
                rectangular_sum(scalar_mul_fn_2(c, f), k.suc, n2) = rectangular_sum(scalar_mul_fn_2(c, f), k, n2) + (c * row_sum(f, n2, k))
                rectangular_sum(scalar_mul_fn_2(c, f), k.suc, n2) = (c * rectangular_sum(f, k, n2)) + (c * row_sum(f, n2, k))
                rectangular_sum(f, k.suc, n2) = rectangular_sum(f, k, n2) + row_sum(f, n2, k)
                rectangular_sum(scalar_mul_fn_2(c, f), k.suc, n2) = c * (rectangular_sum(f, k, n2) + row_sum(f, n2, k))
                rectangular_sum(scalar_mul_fn_2(c, f), k.suc, n2) = c * rectangular_sum(f, k.suc, n2)
            }
            p(k.suc)
        }
    }
    p(m)
}

/// Sum of ones over a row equals the number of columns.
theorem row_sum_const_one(n: Nat, i: Nat) {
    row_sum(const_fn_2(Real.1), n, i) = nat_to_real(n)
} by {
    define p(k: Nat) -> Bool {
        row_sum(const_fn_2(Real.1), k, i) = nat_to_real(k)
    }

    partial[Real](const_fn_2(Real.1, i), Nat.0) = row_sum(const_fn_2(Real.1), Nat.0, i)
    partial[Real](const_fn_2(Real.1, i), Nat.0) = Real.0
    p(Nat.0)

    forall(k: Nat) {
        if p(k) {
            row_sum(const_fn_2(Real.1), k.suc, i) = row_sum(const_fn_2(Real.1), k, i) + const_fn_2(Real.1, i, k)
            const_fn_2(Real.1, i, k) = Real.1
            row_sum(const_fn_2(Real.1), k, i) = nat_to_real(k)
            nat_to_real(k.suc) = nat_to_real(k) + Real.1
            row_sum(const_fn_2(Real.1), k.suc, i) = nat_to_real(k.suc)
            p(k.suc)
        }
    }
    p(n)
}

/// Rectangular sum of ones counts the number of lattice points.
theorem rectangular_sum_const_one(m: Nat, n: Nat) {
    rectangular_sum(const_fn_2(Real.1), m, n) = nat_to_real(m) * nat_to_real(n)
} by {
    define p(k: Nat) -> Bool {
        rectangular_sum(const_fn_2(Real.1), k, n) = nat_to_real(k) * nat_to_real(n)
    }

    p(Nat.0)

    forall(k: Nat) {
        if p(k) {
            rectangular_sum(const_fn_2(Real.1), k.suc, n) = rectangular_sum(const_fn_2(Real.1), k, n) + row_sum(const_fn_2(Real.1), n, k)
            rectangular_sum(const_fn_2(Real.1), k, n) = nat_to_real(k) * nat_to_real(n)
            row_sum(const_fn_2(Real.1), n, k) = nat_to_real(n)
            nat_to_real(n) = Real.1 * nat_to_real(n)
            nat_to_real(k.suc) = nat_to_real(k) + Real.1
            rectangular_sum(const_fn_2(Real.1), k.suc, n) = nat_to_real(k.suc) * nat_to_real(n)
            p(k.suc)
        }
    }
    p(m)
}

/// Rectangular sum of a constant equals the product of the area and the constant.
theorem rectangular_sum_const(c: Real, m: Nat, n: Nat) {
    rectangular_sum(const_fn_2(c), m, n) = nat_to_real(m) * nat_to_real(n) * c
} by {
    forall(i: Nat, j: Nat) {
        const_fn_2(c, i, j) = c
        const_fn_2(Real.1, i, j) = Real.1
        scalar_mul_fn_2(c, const_fn_2(Real.1), i, j) = c * const_fn_2(Real.1, i, j)
        c * Real.1 = c
        const_fn_2(c, i, j) = scalar_mul_fn_2(c, const_fn_2(Real.1), i, j)
    }
    const_fn_2(c) = scalar_mul_fn_2(c, const_fn_2(Real.1))
    rectangular_sum(const_fn_2(c), m, n) = rectangular_sum(scalar_mul_fn_2(c, const_fn_2(Real.1)), m, n)
    rectangular_sum(scalar_mul_fn_2(c, const_fn_2(Real.1)), m, n) = c * rectangular_sum(const_fn_2(Real.1), m, n)
    rectangular_sum(const_fn_2(Real.1), m, n) = nat_to_real(m) * nat_to_real(n)
    c * rectangular_sum(const_fn_2(Real.1), m, n) = c * (nat_to_real(m) * nat_to_real(n))
    (c * nat_to_real(m)) * nat_to_real(n) = c * (nat_to_real(m) * nat_to_real(n))
    c * nat_to_real(m) * nat_to_real(n) = c * (nat_to_real(m) * nat_to_real(n))
    c * (nat_to_real(m) * nat_to_real(n)) = (nat_to_real(m) * nat_to_real(n)) * c
    (nat_to_real(m) * nat_to_real(n)) * c = nat_to_real(m) * nat_to_real(n) * c
}

/// Rectangular sums of a bounded function are bounded by the area times the bound.
theorem rectangular_sum_upper_bound(f: (Nat, Nat) -> Real, bound: Real, m: Nat, n: Nat) {
    nonneg_fn_2(f) and is_upper_bound_fn_2(f, bound)
    implies
    rectangular_sum(f, m, n) <= nat_to_real(m) * nat_to_real(n) * bound
} by {
    forall(i: Nat, j: Nat) {
        f(i, j) <= bound
        const_fn_2(bound, i, j) = bound
        bound <= const_fn_2(bound, i, j)
        f(i, j) <= const_fn_2(bound, i, j)
    }
    lte_fn_2(f, const_fn_2(bound))
    rectangular_sum(f, m, n) <= rectangular_sum(const_fn_2(bound), m, n)
    rectangular_sum(const_fn_2(bound), m, n) = nat_to_real(m) * nat_to_real(n) * bound
}

/// Rectangular sums of a function bounded below are bounded below by the area times the bound.
theorem rectangular_sum_lower_bound(f: (Nat, Nat) -> Real, bound: Real, m: Nat, n: Nat) {
    is_lower_bound_fn_2(f, bound)
    implies
    rectangular_sum(f, m, n) >= nat_to_real(m) * nat_to_real(n) * bound
} by {
    if is_lower_bound_fn_2(f, bound) {
        forall(i: Nat, j: Nat) {
            f(i, j) >= bound
            bound <= f(i, j)
            const_fn_2(bound, i, j) = bound
            const_fn_2(bound, i, j) <= f(i, j)
        }
        lte_fn_2(const_fn_2(bound), f)
        rectangular_sum(const_fn_2(bound), m, n) <= rectangular_sum(f, m, n)
        rectangular_sum(f, m, n) >= rectangular_sum(const_fn_2(bound), m, n)
        rectangular_sum(const_fn_2(bound), m, n) = nat_to_real(m) * nat_to_real(n) * bound
        rectangular_sum(f, m, n) >= nat_to_real(m) * nat_to_real(n) * bound
    }
}

/// Rectangular sum is distributive over subtraction.
theorem rectangular_sum_sub(f: (Nat, Nat) -> Real, g: (Nat, Nat) -> Real, m: Nat, n: Nat) {
    rectangular_sum(sub_fn_2(f, g), m, n)
    =
    rectangular_sum(f, m, n) - rectangular_sum(g, m, n)
} by {
    define neg_g(i: Nat, j: Nat) -> Real {
        Real.0 - g(i, j)
    }

    forall(i: Nat, j: Nat) {
        neg_g(i, j) = Real.0 - g(i, j)
        sub_fn_2(f, g, i, j) = f(i, j) - g(i, j)
        add_fn_2(f, neg_g, i, j) = f(i, j) + neg_g(i, j)
        f(i, j) + neg_g(i, j) = f(i, j) + (Real.0 - g(i, j))
        f(i, j) + (Real.0 - g(i, j)) = f(i, j) - g(i, j)
        sub_fn_2(f, g, i, j) = add_fn_2(f, neg_g, i, j)
    }
    sub_fn_2(f, g) = add_fn_2(f, neg_g)
    rectangular_sum(sub_fn_2(f, g), m, n) = rectangular_sum(add_fn_2(f, neg_g), m, n)
    rectangular_sum(add_fn_2(f, neg_g), m, n) = rectangular_sum(f, m, n) + rectangular_sum(neg_g, m, n)

    forall(i: Nat, j: Nat) {
        neg_g(i, j) = Real.0 - g(i, j)
        scalar_mul_fn_2(Real.0 - Real.1, g, i, j) = (Real.0 - Real.1) * g(i, j)
        (Real.0 - Real.1) * g(i, j) = Real.0 - g(i, j)
        neg_g(i, j) = scalar_mul_fn_2(Real.0 - Real.1, g, i, j)
    }
    neg_g = scalar_mul_fn_2(Real.0 - Real.1, g)
    rectangular_sum(neg_g, m, n) = rectangular_sum(scalar_mul_fn_2(Real.0 - Real.1, g), m, n)
    rectangular_sum(scalar_mul_fn_2(Real.0 - Real.1, g), m, n) = (Real.0 - Real.1) * rectangular_sum(g, m, n)
    (Real.0 - Real.1) * rectangular_sum(g, m, n) = Real.0 - rectangular_sum(g, m, n)
    rectangular_sum(neg_g, m, n) = Real.0 - rectangular_sum(g, m, n)

    rectangular_sum(sub_fn_2(f, g), m, n) = rectangular_sum(f, m, n) + (Real.0 - rectangular_sum(g, m, n))
    rectangular_sum(f, m, n) + (Real.0 - rectangular_sum(g, m, n)) = rectangular_sum(f, m, n) - rectangular_sum(g, m, n)
}

/// Extending a rectangle by one row.
theorem rectangular_sum_extend_row(f: (Nat, Nat) -> Real, m: Nat, n: Nat) {
    rectangular_sum(f, m.suc, n) = rectangular_sum(f, m, n) + row_sum(f, n, m)
}

/// Extending a rectangle by one column.
theorem rectangular_sum_extend_col(f: (Nat, Nat) -> Real, m: Nat, n: Nat) {
    rectangular_sum(f, m, n.suc) = rectangular_sum(f, m, n) + col_sum(f, m, n)
} by {
    rectangular_sum(f, m, n.suc) = rectangular_sum_col_first(f, m, n.suc)
    rectangular_sum(f, m, n) = rectangular_sum_col_first(f, m, n)
}

/// Rectangular sum over increasing dimensions is monotone for nonnegative functions.
theorem rectangular_sum_increasing_rows(f: (Nat, Nat) -> Real, m: Nat, n: Nat) {
    nonneg_fn_2(f)
    implies
    rectangular_sum(f, m, n) <= rectangular_sum(f, m.suc, n)
} by {
    if nonneg_fn_2(f) {
        // Prove row_sum(f, n, m) >= 0
        define r(n3: Nat) -> Bool {
            row_sum(f, n3, m) >= Real.0
        }

        row_sum(f, Nat.0, m) = partial(f(m), Nat.0)
        partial(f(m), Nat.0) = Real.0
        row_sum(f, Nat.0, m) = Real.0
        Real.0 >= Real.0
        row_sum(f, Nat.0, m) >= Real.0
        r(Nat.0)

        forall(n3: Nat) {
            if r(n3) {
                row_sum(f, n3, m) >= Real.0
                f(m, n3) >= Real.0
                Real.0 + Real.0 <= row_sum(f, n3, m) + f(m, n3)
                row_sum(f, n3, m) + f(m, n3) >= Real.0
                row_sum(f, n3.suc, m) = row_sum(f, n3, m) + f(m, n3)
                row_sum(f, n3.suc, m) >= Real.0
                r(n3.suc)
            }
        }

        row_sum(f, n, m) >= Real.0
        Real.0 <= row_sum(f, n, m)
        rectangular_sum(f, m, n) <= rectangular_sum(f, m, n) + row_sum(f, n, m)
        rectangular_sum(f, m, n) + row_sum(f, n, m) >= rectangular_sum(f, m, n)
        rectangular_sum(f, m.suc, n) = rectangular_sum(f, m, n) + row_sum(f, n, m)
        rectangular_sum(f, m.suc, n) >= rectangular_sum(f, m, n)
    }
}

/// Rectangular sums are increasing as the row bound grows.
theorem rectangular_sum_rows_increasing(f: (Nat, Nat) -> Real, n: Nat) {
    nonneg_fn_2(f)
    implies
    is_increasing(rectangular_sum_by_rows(f, n))
} by {
    if nonneg_fn_2(f) {
        forall(m: Nat) {
            rectangular_sum_by_rows(f, n, m) = rectangular_sum(f, m, n)
            rectangular_sum_by_rows(f, n, m.suc) = rectangular_sum(f, m.suc, n)
            rectangular_sum(f, m, n) <= rectangular_sum(f, m.suc, n)
            rectangular_sum_by_rows(f, n, m) <= rectangular_sum_by_rows(f, n, m.suc)
        }
    }
}

/// Rectangular sums are monotone as the row bound grows.
theorem rectangular_sum_rows_monotone(f: (Nat, Nat) -> Real, n: Nat) {
    nonneg_fn_2(f)
    implies
    is_monotone(rectangular_sum_by_rows(f, n))
} by {
    if nonneg_fn_2(f) {
        rectangular_sum_rows_increasing(f, n)
        is_increasing(rectangular_sum_by_rows(f, n))
        increasing_is_monotone(rectangular_sum_by_rows(f, n))
        is_monotone(rectangular_sum_by_rows(f, n))
    }
}

theorem rectangular_sum_increasing_cols(f: (Nat, Nat) -> Real, m: Nat, n: Nat) {
    nonneg_fn_2(f)
    implies
    rectangular_sum(f, m, n) <= rectangular_sum(f, m, n.suc)
} by {
    if nonneg_fn_2(f) {
        // Prove col_sum(f, m, n) >= 0
        define c(m3: Nat) -> Bool {
            col_sum(f, m3, n) >= Real.0
        }

        col_sum(f, Nat.0, n) = partial(flip(f, n), Nat.0)
        partial(flip(f, n), Nat.0) = Real.0
        col_sum(f, Nat.0, n) = Real.0
        Real.0 >= Real.0
        col_sum(f, Nat.0, n) >= Real.0
        c(Nat.0)

        forall(m3: Nat) {
            if c(m3) {
                f(m3, n) >= Real.0
                Real.0 + Real.0 <= col_sum(f, m3, n) + f(m3, n)
                col_sum(f, m3, n) + f(m3, n) >= Real.0
                col_sum(f, m3.suc, n) = partial(flip(f, n), m3.suc)
                partial(flip(f, n), m3.suc) = partial(flip(f, n), m3) + flip(f, n)(m3)
                flip(f, n)(m3) = f(m3, n)
                col_sum(f, m3, n) = partial(flip(f, n), m3)
                col_sum(f, m3.suc, n) = col_sum(f, m3, n) + f(m3, n)
                col_sum(f, m3.suc, n) >= Real.0
                c(m3.suc)
            }
        }

        col_sum(f, m, n) >= Real.0
        rectangular_sum(f, m, n) = rectangular_sum_col_first(f, m, n)
        rectangular_sum(f, m, n.suc) = rectangular_sum_col_first(f, m, n.suc)
        rectangular_sum_col_first(f, m, n.suc) = rectangular_sum_col_first(f, m, n) + col_sum(f, m, n)
        Real.0 <= col_sum(f, m, n)
        rectangular_sum(f, m, n) <= rectangular_sum(f, m, n) + col_sum(f, m, n)
        rectangular_sum(f, m, n) + col_sum(f, m, n) >= rectangular_sum(f, m, n)
        rectangular_sum(f, m, n.suc) >= rectangular_sum(f, m, n)
    }
}

/// Rectangular sums are increasing as the column bound grows.
theorem rectangular_sum_cols_increasing(f: (Nat, Nat) -> Real, m: Nat) {
    nonneg_fn_2(f)
    implies
    is_increasing(rectangular_sum_by_cols(f, m))
} by {
    if nonneg_fn_2(f) {
        forall(n: Nat) {
            rectangular_sum_by_cols(f, m, n) = rectangular_sum(f, m, n)
            rectangular_sum_by_cols(f, m, n.suc) = rectangular_sum(f, m, n.suc)
            rectangular_sum(f, m, n) <= rectangular_sum(f, m, n.suc)
            rectangular_sum_by_cols(f, m, n) <= rectangular_sum_by_cols(f, m, n.suc)
        }
    }
}

/// Rectangular sums are monotone as the column bound grows.
theorem rectangular_sum_cols_monotone(f: (Nat, Nat) -> Real, m: Nat) {
    nonneg_fn_2(f)
    implies
    is_monotone(rectangular_sum_by_cols(f, m))
} by {
    if nonneg_fn_2(f) {
        rectangular_sum_cols_increasing(f, m)
        is_increasing(rectangular_sum_by_cols(f, m))
        increasing_is_monotone(rectangular_sum_by_cols(f, m))
        is_monotone(rectangular_sum_by_cols(f, m))
    }
}

/// Product of two sequences viewed as a rectangular sum.
define product_sum(a: Nat -> Real, b: Nat -> Real, m: Nat, n: Nat) -> Real {
    rectangular_sum(prod_fn(a, b), m, n)
}

/// Product sum factors as product of partial sums.
theorem product_sum_factors(a: Nat -> Real, b: Nat -> Real, m: Nat, n: Nat) {
    product_sum(a, b, m, n) = partial(a, m) * partial(b, n)
} by {
    define p(k: Nat) -> Bool {
        product_sum(a, b, k, n) = partial(a, k) * partial(b, n)
    }
    product_sum(a, b, Nat.0, n) = rectangular_sum(prod_fn(a, b), Nat.0, n)
    rectangular_sum(prod_fn(a, b), Nat.0, n) = Real.0
    partial(a, Nat.0) = Real.0
    Real.0 * partial(b, n) = Real.0
    product_sum(a, b, Nat.0, n) = partial(a, Nat.0) * partial(b, n)
    p(Nat.0)

    forall(k: Nat) {
        if p(k) {
            define r(n3: Nat) -> Bool {
                row_sum(prod_fn(a, b), n3, k) = a(k) * partial(b, n3)
            }
            row_sum(prod_fn(a, b), Nat.0, k) = partial(prod_fn(a, b, k), Nat.0)
            partial(prod_fn(a, b, k), Nat.0) = Real.0
            row_sum(prod_fn(a, b), Nat.0, k) = Real.0
            partial(b, Nat.0) = Real.0
            a(k) * Real.0 = Real.0
            a(k) * partial(b, Nat.0) = Real.0
            row_sum(prod_fn(a, b), Nat.0, k) = a(k) * partial(b, Nat.0)
            r(Nat.0)

            forall(n3: Nat) {
                if r(n3) {
                    row_sum(prod_fn(a, b), n3.suc, k) = row_sum(prod_fn(a, b), n3, k) + prod_fn(a, b, k, n3)
                    prod_fn(a, b, k, n3) = a(k) * b(n3)
                    partial(b, n3.suc) = partial(b, n3) + b(n3)
                    row_sum(prod_fn(a, b), n3.suc, k) = a(k) * partial(b, n3.suc)
                    r(n3.suc)
                }
            }

            r(n)
            product_sum(a, b, k.suc, n) = product_sum(a, b, k, n) + row_sum(prod_fn(a, b), n, k)
            product_sum(a, b, k, n) = partial(a, k) * partial(b, n)
            row_sum(prod_fn(a, b), n, k) = a(k) * partial(b, n)
            product_sum(a, b, k.suc, n) = partial(a, k) * partial(b, n) + a(k) * partial(b, n)
            partial(a, k.suc) = partial(a, k) + a(k)
            partial(a, k.suc) * partial(b, n) = (partial(a, k) + a(k)) * partial(b, n)
            (partial(a, k) + a(k)) * partial(b, n) = partial(a, k) * partial(b, n) + a(k) * partial(b, n)
            product_sum(a, b, k.suc, n) = partial(a, k.suc) * partial(b, n)
            p(k.suc)
        }
    }

    p(m)
}

/// Splitting a rectangular sum horizontally.
theorem rectangular_sum_split_rows(f: (Nat, Nat) -> Real, m1: Nat, m2: Nat, n: Nat) {
    rectangular_sum(f, m1 + m2, n)
    =
    rectangular_sum(f, m1, n) + rectangular_sum(shift_rows(f, m1), m2, n)
} by {
    define p(k: Nat) -> Bool {
        rectangular_sum(f, m1 + k, n)
        =
        rectangular_sum(f, m1, n) + rectangular_sum(shift_rows(f, m1), k, n)
    }
    rectangular_sum(shift_rows(f, m1), Nat.0, n) = Real.0
    rectangular_sum(f, m1 + Nat.0, n) = rectangular_sum(f, m1, n) + Real.0
    rectangular_sum(f, m1 + Nat.0, n) = rectangular_sum(f, m1, n) + rectangular_sum(shift_rows(f, m1), Nat.0, n)
    p(Nat.0)

    forall(k: Nat) {
        if p(k) {
            define r(n3: Nat) -> Bool {
                row_sum(shift_rows(f, m1), n3, k) = row_sum(f, n3, m1 + k)
            }
            row_sum(shift_rows(f, m1), Nat.0, k) = partial(shift_rows(f, m1, k), Nat.0)
            partial(shift_rows(f, m1, k), Nat.0) = Real.0
            row_sum(shift_rows(f, m1), Nat.0, k) = Real.0
            row_sum(f, Nat.0, m1 + k) = partial(f(m1 + k), Nat.0)
            partial(f(m1 + k), Nat.0) = Real.0
            row_sum(f, Nat.0, m1 + k) = Real.0
            row_sum(shift_rows(f, m1), Nat.0, k) = row_sum(f, Nat.0, m1 + k)
            r(Nat.0)

            forall(n3: Nat) {
                if r(n3) {
                    row_sum(shift_rows(f, m1), n3, k) = row_sum(f, n3, m1 + k)
                    row_sum(shift_rows(f, m1), n3.suc, k) = row_sum(shift_rows(f, m1), n3, k) + shift_rows(f, m1, k, n3)
                    shift_rows(f, m1, k, n3) = f(m1 + k, n3)
                    row_sum(f, n3.suc, m1 + k) = row_sum(f, n3, m1 + k) + f(m1 + k, n3)
                    row_sum(shift_rows(f, m1), n3.suc, k) = row_sum(f, n3.suc, m1 + k)
                    r(n3.suc)
                }
            }

            r(n)
            rectangular_sum(shift_rows(f, m1), k.suc, n) = rectangular_sum(shift_rows(f, m1), k, n) + row_sum(shift_rows(f, m1), n, k)
            row_sum(shift_rows(f, m1), n, k) = row_sum(f, n, m1 + k)
            rectangular_sum(shift_rows(f, m1), k, n) + row_sum(shift_rows(f, m1), n, k) = rectangular_sum(shift_rows(f, m1), k, n) + row_sum(f, n, m1 + k)
            rectangular_sum(shift_rows(f, m1), k.suc, n) = rectangular_sum(shift_rows(f, m1), k, n) + row_sum(f, n, m1 + k)
            rectangular_sum(f, m1 + k, n) = rectangular_sum(f, m1, n) + rectangular_sum(shift_rows(f, m1), k, n)
            rectangular_sum(f, m1 + k.suc, n) = rectangular_sum(f, m1 + k, n) + row_sum(f, n, m1 + k)
            rectangular_sum(f, m1 + k.suc, n) = (rectangular_sum(f, m1, n) + rectangular_sum(shift_rows(f, m1), k, n)) + row_sum(f, n, m1 + k)
            rectangular_sum(f, m1 + k.suc, n) = rectangular_sum(f, m1, n) + (rectangular_sum(shift_rows(f, m1), k, n) + row_sum(f, n, m1 + k))
            rectangular_sum(f, m1 + k.suc, n) = rectangular_sum(f, m1, n) + rectangular_sum(shift_rows(f, m1), k.suc, n)
            p(k.suc)
        }
    }

    p(m2)
}

/// Splitting a rectangular sum vertically.
theorem rectangular_sum_split_cols(f: (Nat, Nat) -> Real, m: Nat, n1: Nat, n2: Nat) {
    rectangular_sum(f, m, n1 + n2)
    =
    rectangular_sum(f, m, n1) + rectangular_sum(shift_cols(f, n1), m, n2)
} by {
    // Convert everything to col_first form
    rectangular_sum(f, m, n1 + n2) = rectangular_sum_col_first(f, m, n1 + n2)
    rectangular_sum(f, m, n1) = rectangular_sum_col_first(f, m, n1)
    rectangular_sum(shift_cols(f, n1), m, n2) = rectangular_sum_col_first(shift_cols(f, n1), m, n2)

    // Prove by induction on n2
    define p(k: Nat) -> Bool {
        rectangular_sum_col_first(f, m, n1 + k)
        =
        rectangular_sum_col_first(f, m, n1) + rectangular_sum_col_first(shift_cols(f, n1), m, k)
    }

    rectangular_sum_col_first(shift_cols(f, n1), m, Nat.0) = Real.0
    rectangular_sum_col_first(f, m, n1 + Nat.0) = rectangular_sum_col_first(f, m, n1) + Real.0
    rectangular_sum_col_first(f, m, n1 + Nat.0) = rectangular_sum_col_first(f, m, n1) + rectangular_sum_col_first(shift_cols(f, n1), m, Nat.0)
    p(Nat.0)

    forall(k: Nat) {
        if p(k) {
            define r(m3: Nat) -> Bool {
                col_sum(shift_cols(f, n1), m3, k) = col_sum(f, m3, n1 + k)
            }
            col_sum(shift_cols(f, n1), Nat.0, k) = Real.0
            col_sum(f, Nat.0, n1 + k) = Real.0
            r(Nat.0)

            forall(m3: Nat) {
                if r(m3) {
                    col_sum(shift_cols(f, n1), m3.suc, k) = partial(flip(shift_cols(f, n1), k), m3.suc)
                    partial(flip(shift_cols(f, n1), k), m3.suc) = partial(flip(shift_cols(f, n1), k), m3) + flip(shift_cols(f, n1), k)(m3)
                    flip(shift_cols(f, n1), k)(m3) = shift_cols(f, n1, m3, k)
                    col_sum(shift_cols(f, n1), m3, k) = partial(flip(shift_cols(f, n1), k), m3)
                    col_sum(shift_cols(f, n1), m3.suc, k) = col_sum(shift_cols(f, n1), m3, k) + shift_cols(f, n1, m3, k)
                    shift_cols(f, n1, m3, k) = f(m3, n1 + k)
                    col_sum(f, m3.suc, n1 + k) = partial(flip(f, n1 + k), m3.suc)
                    partial(flip(f, n1 + k), m3.suc) = partial(flip(f, n1 + k), m3) + flip(f, n1 + k)(m3)
                    flip(f, n1 + k)(m3) = f(m3, n1 + k)
                    col_sum(f, m3, n1 + k) = partial(flip(f, n1 + k), m3)
                    col_sum(f, m3.suc, n1 + k) = col_sum(f, m3, n1 + k) + f(m3, n1 + k)
                    col_sum(shift_cols(f, n1), m3, k) + shift_cols(f, n1, m3, k) = col_sum(f, m3, n1 + k) + shift_cols(f, n1, m3, k)
                    col_sum(f, m3, n1 + k) + shift_cols(f, n1, m3, k) = col_sum(f, m3, n1 + k) + f(m3, n1 + k)
                    col_sum(shift_cols(f, n1), m3.suc, k) = col_sum(f, m3, n1 + k) + f(m3, n1 + k)
                    col_sum(shift_cols(f, n1), m3.suc, k) = col_sum(f, m3.suc, n1 + k)
                    r(m3.suc)
                }
            }

            r(m)
            col_sum(shift_cols(f, n1), m, k) = col_sum(f, m, n1 + k)
            rectangular_sum_col_first(shift_cols(f, n1), m, k.suc) = rectangular_sum_col_first(shift_cols(f, n1), m, k) + col_sum(shift_cols(f, n1), m, k)
            rectangular_sum_col_first(shift_cols(f, n1), m, k) + col_sum(shift_cols(f, n1), m, k) = rectangular_sum_col_first(shift_cols(f, n1), m, k) + col_sum(f, m, n1 + k)
            rectangular_sum_col_first(shift_cols(f, n1), m, k.suc) = rectangular_sum_col_first(shift_cols(f, n1), m, k) + col_sum(f, m, n1 + k)
            n1 + k.suc = (n1 + k).suc
            rectangular_sum_col_first(f, m, n1 + k) = rectangular_sum(f, m, n1 + k)
            rectangular_sum_col_first(f, m, n1 + k.suc) = rectangular_sum(f, m, n1 + k.suc)
            rectangular_sum(f, m, n1 + k) + col_sum(f, m, n1 + k) = rectangular_sum(f, m, (n1 + k).suc)
            rectangular_sum_col_first(f, m, n1 + k.suc) = rectangular_sum_col_first(f, m, n1 + k) + col_sum(f, m, n1 + k)
            rectangular_sum_col_first(f, m, n1 + k) = rectangular_sum_col_first(f, m, n1) + rectangular_sum_col_first(shift_cols(f, n1), m, k)
            rectangular_sum_col_first(f, m, n1 + k.suc) = (rectangular_sum_col_first(f, m, n1) + rectangular_sum_col_first(shift_cols(f, n1), m, k)) + col_sum(f, m, n1 + k)
            rectangular_sum_col_first(f, m, n1 + k.suc) = rectangular_sum_col_first(f, m, n1) + (rectangular_sum_col_first(shift_cols(f, n1), m, k) + col_sum(f, m, n1 + k))
            rectangular_sum_col_first(f, m, n1 + k.suc) = rectangular_sum_col_first(f, m, n1) + rectangular_sum_col_first(shift_cols(f, n1), m, k.suc)
            p(k.suc)
        }
    }

    p(n2)
    rectangular_sum_col_first(f, m, n1 + n2) = rectangular_sum_col_first(f, m, n1) + rectangular_sum_col_first(shift_cols(f, n1), m, n2)
    rectangular_sum(f, m, n1 + n2) = rectangular_sum(f, m, n1) + rectangular_sum(shift_cols(f, n1), m, n2)
}

/// Composing row shifts.
theorem shift_rows_compose(f: (Nat, Nat) -> Real, k1: Nat, k2: Nat, i: Nat, j: Nat) {
    shift_rows(shift_rows(f, k1), k2, i, j) = shift_rows(f, k1 + k2, i, j)
}

/// Composing column shifts.
theorem shift_cols_compose(f: (Nat, Nat) -> Real, k1: Nat, k2: Nat, i: Nat, j: Nat) {
    shift_cols(shift_cols(f, k1), k2, i, j) = shift_cols(f, k1 + k2, i, j)
}

/// Row shift commutes with pointwise addition.
theorem shift_rows_add(f: (Nat, Nat) -> Real, g: (Nat, Nat) -> Real, k: Nat, i: Nat, j: Nat) {
    shift_rows(add_fn_2(f, g), k, i, j) = add_fn_2(shift_rows(f, k), shift_rows(g, k), i, j)
}

/// Column shift commutes with pointwise addition.
theorem shift_cols_add(f: (Nat, Nat) -> Real, g: (Nat, Nat) -> Real, k: Nat, i: Nat, j: Nat) {
    shift_cols(add_fn_2(f, g), k, i, j) = add_fn_2(shift_cols(f, k), shift_cols(g, k), i, j)
}

/// Row shift commutes with scalar multiplication.
theorem shift_rows_scale(c: Real, f: (Nat, Nat) -> Real, k: Nat, i: Nat, j: Nat) {
    shift_rows(scalar_mul_fn_2(c, f), k, i, j) = scalar_mul_fn_2(c, shift_rows(f, k), i, j)
}

/// Column shift commutes with scalar multiplication.
theorem shift_cols_scale(c: Real, f: (Nat, Nat) -> Real, k: Nat, i: Nat, j: Nat) {
    shift_cols(scalar_mul_fn_2(c, f), k, i, j) = scalar_mul_fn_2(c, shift_cols(f, k), i, j)
}

/// Row and column shifts commute.
theorem shift_rows_cols_commute(f: (Nat, Nat) -> Real, k1: Nat, k2: Nat, i: Nat, j: Nat) {
    shift_rows(shift_cols(f, k2), k1, i, j) = shift_cols(shift_rows(f, k1), k2, i, j)
}

/// Alias endpoint: a rectangle with zero row count has zero sum.
theorem rectangular_sum_zero_height(f: (Nat, Nat) -> Real, n: Nat) {
    rectangular_sum(f, Nat.0, n) = Real.0
} by {
    rectangular_sum_zero_rows(f, n)
}

/// Alias endpoint: a rectangle with zero column count has zero sum.
theorem rectangular_sum_zero_width(f: (Nat, Nat) -> Real, m: Nat) {
    rectangular_sum(f, m, Nat.0) = Real.0
} by {
    rectangular_sum_zero_cols(f, m)
}

/// Alias endpoint: extending the height appends the next row sum.
theorem rectangular_sum_succ_rows(f: (Nat, Nat) -> Real, m: Nat, n: Nat) {
    rectangular_sum(f, m.suc, n) = rectangular_sum(f, m, n) + row_sum(f, n, m)
} by {
    rectangular_sum_extend_row(f, m, n)
}

/// Alias endpoint: extending the width appends the next column sum.
theorem rectangular_sum_succ_cols(f: (Nat, Nat) -> Real, m: Nat, n: Nat) {
    rectangular_sum(f, m, n.suc) = rectangular_sum(f, m, n) + col_sum(f, m, n)
} by {
    rectangular_sum_extend_col(f, m, n)
}

/// Alias endpoint: rectangular summation is additive pointwise.
theorem rectangular_sum_add_distrib(f: (Nat, Nat) -> Real, g: (Nat, Nat) -> Real, m: Nat, n: Nat) {
    rectangular_sum(add_fn_2(f, g), m, n) = rectangular_sum(f, m, n) + rectangular_sum(g, m, n)
} by {
    rectangular_sum_add(f, g, m, n)
}

/// Alias endpoint: rectangular summation commutes with left scalar multiplication.
theorem rectangular_sum_scale_left(c: Real, f: (Nat, Nat) -> Real, m: Nat, n: Nat) {
    rectangular_sum(scalar_mul_fn_2(c, f), m, n) = c * rectangular_sum(f, m, n)
} by {
    rectangular_sum_scale(c, f, m, n)
}

/// Alias endpoint: pointwise order transports to rectangular finite sums.
theorem rectangular_sum_le_of_lte_fn(f: (Nat, Nat) -> Real, g: (Nat, Nat) -> Real, m: Nat, n: Nat) {
    lte_fn_2(f, g) implies rectangular_sum(f, m, n) <= rectangular_sum(g, m, n)
} by {
    rectangular_sum_monotone(f, g, m, n)
}

/// Alias endpoint: rectangular finite sums are bounded above by an area-scaled bound.
theorem rectangular_sum_bound_above(f: (Nat, Nat) -> Real, bound: Real, m: Nat, n: Nat) {
    nonneg_fn_2(f) and is_upper_bound_fn_2(f, bound)
    implies
    rectangular_sum(f, m, n) <= nat_to_real(m) * nat_to_real(n) * bound
} by {
    rectangular_sum_upper_bound(f, bound, m, n)
}

/// Alias endpoint: rectangular finite sums are bounded below by an area-scaled bound.
theorem rectangular_sum_bound_below(f: (Nat, Nat) -> Real, bound: Real, m: Nat, n: Nat) {
    is_lower_bound_fn_2(f, bound)
    implies
    rectangular_sum(f, m, n) >= nat_to_real(m) * nat_to_real(n) * bound
} by {
    rectangular_sum_lower_bound(f, bound, m, n)
}

/// Alias endpoint: split a rectangle along the row axis.
theorem rectangular_sum_rows_split_at(f: (Nat, Nat) -> Real, m1: Nat, m2: Nat, n: Nat) {
    rectangular_sum(f, m1 + m2, n)
    =
    rectangular_sum(f, m1, n) + rectangular_sum(shift_rows(f, m1), m2, n)
} by {
    rectangular_sum_split_rows(f, m1, m2, n)
}

/// Alias endpoint: split a rectangle along the column axis.
theorem rectangular_sum_cols_split_at(f: (Nat, Nat) -> Real, m: Nat, n1: Nat, n2: Nat) {
    rectangular_sum(f, m, n1 + n2)
    =
    rectangular_sum(f, m, n1) + rectangular_sum(shift_cols(f, n1), m, n2)
} by {
    rectangular_sum_split_cols(f, m, n1, n2)
}
