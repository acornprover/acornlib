from data.basic.set import Set, all_sets_subset_universal, double_inclusion, intersection_contains_intro,
    universal_set_compl_is_empty
from real.real_field import Real
from real.topology import closure, interior
from real.topology_boundary import boundary, boundary_contains_eq, exterior,
    exterior_eq_closure_complement
from real.topology_dense import dense_real_set_intro, is_dense_real_set

/// True if the complement of a real set is dense.
define is_codense_real_set(s: Set[Real]) -> Bool {
    is_dense_real_set(s.c)
}

/// Codensity is density of the complement.
theorem codense_real_set_eq_dense_complement(s: Set[Real]) {
    is_codense_real_set(s) = is_dense_real_set(s.c)
}

/// A dense real set has empty exterior.
theorem exterior_of_dense_real_set_is_empty(s: Set[Real]) {
    is_dense_real_set(s) implies exterior(s) = Set[Real].empty_set
} by {
    if is_dense_real_set(s) {
        exterior_eq_closure_complement(s)
        exterior(s) = closure(s).c
        closure(s) = Set[Real].universal_set
        exterior(s) = Set[Real].universal_set.c
        universal_set_compl_is_empty[Real]
        Set[Real].universal_set.c = Set[Real].empty_set
        exterior(s) = Set[Real].empty_set
    }
}

/// A dense real set has complement with empty interior.
theorem interior_complement_of_dense_real_set_is_empty(s: Set[Real]) {
    is_dense_real_set(s) implies interior(s.c) = Set[Real].empty_set
} by {
    if is_dense_real_set(s) {
        exterior_of_dense_real_set_is_empty(s)
        exterior(s) = Set[Real].empty_set
        exterior(s) = interior(s.c)
        interior(s.c) = Set[Real].empty_set
    }
}

/// A codense real set has empty interior.
theorem interior_of_codense_real_set_is_empty(s: Set[Real]) {
    is_codense_real_set(s) implies interior(s) = Set[Real].empty_set
} by {
    if is_codense_real_set(s) {
        is_dense_real_set(s.c)
        interior_complement_of_dense_real_set_is_empty(s.c)
        interior(s.c.c) = Set[Real].empty_set
from data.basic.set import compl_of_compl_is_self
        compl_of_compl_is_self[Real](s)
        s.c.c = s
        interior(s) = Set[Real].empty_set
    }
}

/// The empty real set is codense.
theorem empty_real_set_is_codense {
    is_codense_real_set(Set[Real].empty_set)
} by {
from data.basic.set import empty_set_compl_is_universal
    empty_set_compl_is_universal[Real]
    Set[Real].empty_set.c = Set[Real].universal_set
    forall(x: Real) {
        closure(Set[Real].universal_set).contains(x)
    }
    dense_real_set_intro(Set[Real].universal_set)
    is_dense_real_set(Set[Real].universal_set)
    is_dense_real_set(Set[Real].empty_set.c)
    is_codense_real_set(Set[Real].empty_set)
}

/// A dense and codense real set has universal boundary.
theorem boundary_of_dense_codense_real_set_is_universal(s: Set[Real]) {
    is_dense_real_set(s) and is_codense_real_set(s) implies boundary(s) = Set[Real].universal_set
} by {
    if is_dense_real_set(s) and is_codense_real_set(s) {
        closure(s) = Set[Real].universal_set
        is_dense_real_set(s.c)
        closure(s.c) = Set[Real].universal_set
        forall(x: Real) {
            if Set[Real].universal_set.contains(x) {
                closure(s).contains(x)
                closure(s.c).contains(x)
                intersection_contains_intro(closure(s), closure(s.c), x)
                boundary(s).contains(x)
            }
        }
        Set[Real].universal_set.subset(boundary(s))
        all_sets_subset_universal[Real](boundary(s))
        boundary(s).subset(Set[Real].universal_set)
        double_inclusion(boundary(s), Set[Real].universal_set)
        boundary(s) = Set[Real].universal_set
    }
}

/// A real set whose boundary is universal is dense.
theorem dense_real_set_of_universal_boundary(s: Set[Real]) {
    boundary(s) = Set[Real].universal_set implies is_dense_real_set(s)
} by {
    if boundary(s) = Set[Real].universal_set {
        forall(x: Real) {
            Set[Real].universal_set.contains(x)
            boundary(s).contains(x)
            boundary_contains_eq(s, x)
            closure(s).contains(x)
        }
        dense_real_set_intro(s)
        is_dense_real_set(s)
    }
}

/// A real set whose boundary is universal is codense.
theorem codense_real_set_of_universal_boundary(s: Set[Real]) {
    boundary(s) = Set[Real].universal_set implies is_codense_real_set(s)
} by {
    if boundary(s) = Set[Real].universal_set {
        forall(x: Real) {
            Set[Real].universal_set.contains(x)
            boundary(s).contains(x)
            boundary_contains_eq(s, x)
            closure(s.c).contains(x)
        }
        dense_real_set_intro(s.c)
        is_dense_real_set(s.c)
        is_codense_real_set(s)
    }
}

/// Universal boundary is equivalent to being both dense and codense.
theorem universal_boundary_eq_dense_and_codense(s: Set[Real]) {
    (boundary(s) = Set[Real].universal_set) = (is_dense_real_set(s) and is_codense_real_set(s))
} by {
    if boundary(s) = Set[Real].universal_set {
        dense_real_set_of_universal_boundary(s)
        codense_real_set_of_universal_boundary(s)
        is_dense_real_set(s) and is_codense_real_set(s)
    }
    if is_dense_real_set(s) and is_codense_real_set(s) {
        boundary_of_dense_codense_real_set_is_universal(s)
        boundary(s) = Set[Real].universal_set
    }
    (boundary(s) = Set[Real].universal_set) = (is_dense_real_set(s) and is_codense_real_set(s))
}
