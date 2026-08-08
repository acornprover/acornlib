from data.basic.functions import compose, function_extensionality, is_bijection, is_injective, is_surjective
from nat import Nat, lte_trans
from list import partial, sum, map, List
from algebra.semigroup import mul_fn
from algebra.add_semigroup import add_fn
from real.real_ring import Real, converges, limit, converges_to, mul_neg_one_left, mul_abs, real_mul_comm
from real.real_seq import limit_add_seq, converges_imp_converges_to, converges_to_imp_converges, converges_to_unique, tail_bound, cauchy_bound, eventual_ub, ub_imp_limit_lte, eventual_lb, lb_lte_limit, abs_lt_imp_close_to_zero
from real.real_series import is_lower_bound, add_seq, const_converges, const_limit, neg_seq, mul_seq, comparison_test, seq_lte, is_upper_bound, tail
from real.real_ring import mul_le_mul_nonneg
from real.limits import vanishes, abs_seq
from real.real_base import add_assoc, add_neg_eq_zero, add_zero_left, add_zero_right, neg_neg, sub_moves_sides, abs_neg, lt_add_pos
from real.prod_seq import prod_seq
from data.basic.set import Set, is_decreasing, is_increasing, seq_intersection, decreasing_seq_transitive, seq_union,
    seq_complement, seq_union_universal_imp_complement_intersection_empty, seq_union_nat_lt_set_universal,
    seq_union_set_image, set_ext, nat_lt_set, nat_lt_set_contains_eq, nat_lt_set_subset_suc,
    nat_lt_set_suc_eq_union, set_image, set_image_monotone, set_image_seq, set_image_union,
    set_image_singleton, set_image_contains_witness, set_image_universal_of_surjective,
    maps_into_set_image, set_is_disjoint_compl, union_compl_is_universal,
    seq_complement_is_decreasing_of_increasing, image_of_range, image_of_range_eq_set_image,
    image_of_range_increasing, image_of_range_union_eq_universal_image, image_of_range_union_universal,
    image_of_range_disjoint_from_next, image_of_range_suc_eq_union_singleton,
    at_least, nat_from, set_lower_bound, nat_from_superset_of_bounded,
    decreasing_empty_intersection_eventually_bounded, decreasing_seq_eventual_lower_bound

numerals Real

// This file defines absolute convergence for series.

/// The absolute value of each element in a sequence.
/// For a sequence a, this returns the sequence n ↦ |a(n)|.
define abs_fn(a: Nat -> Real) -> (Nat -> Real) {
    function(n: Nat) { a(n).abs }
}

/// Filter a sequence by a predicate, zeroing out terms where the predicate is false.
define filter_seq(f: Nat -> Real, p: Nat -> Bool, n: Nat) -> Real {
    if p(n) { f(n) } else { Real.0 }
}

/// A mask of f, including only terms whose indices are in the set s.
define mask(f: Nat -> Real, s: Set[Nat], n: Nat) -> Real {
    filter_seq(f, s.contains, n)
}

/// The sum of the mask for each set in a sequence of sets.
/// For a sequence a and a sequence of sets s, this returns i ↦ ∑ₙ a(n) where n ∈ s(i).
define mask_seq_sum(a: Nat -> Real, s: Nat -> Set[Nat], i: Nat) -> Real {
    limit(partial(mask(a, s(i))))
}

/// A series converges absolutely if the series of absolute values converges.
define absolutely_converges(a: Nat -> Real) -> Bool {
    converges(partial(abs_fn(a)))
}

/// A sequence admits an upper bound if some real number bounds every term from above.
define has_upper_bound_seq(a: Nat -> Real) -> Bool {
    exists(bound: Real) { is_upper_bound(a, bound) }
}

/// The absolute value of any element in the constant zero sequence is zero.
theorem abs_fn_zero(n: Nat) {
    abs_fn(constant[Nat, Real](Real.0))(n) = Real.0
}

/// The absolute value of any element in a sequence is nonnegative.
theorem abs_fn_nonneg(a: Nat -> Real, n: Nat) {
    abs_fn(a)(n) >= Real.0
}

/// Absolute value commutes with scalar multiplication (up to taking abs of scalar).
theorem abs_fn_scalar_mul(c: Real, a: Nat -> Real, n: Nat) {
    abs_fn(mul_fn(c, a))(n) = c.abs * abs_fn(a)(n)
}

/// abs_fn is equivalent to composing with Real.abs.
theorem abs_fn_eq_compose(a: Nat -> Real, n: Nat) {
    abs_fn(a)(n) = compose(Real.abs, a)(n)
}

/// The absolute value of a pointwise product is the product of absolute values.
theorem abs_fn_prod_seq(a: Nat -> Real, b: Nat -> Real, n: Nat) {
    abs_fn(prod_seq(a, b))(n) = abs_fn(a)(n) * abs_fn(b)(n)
} by {
    abs_fn(prod_seq(a, b))(n) = prod_seq(a, b, n).abs
    prod_seq(a, b, n) = a(n) * b(n)
    (a(n) * b(n)).abs = a(n).abs * b(n).abs
    abs_fn(a)(n) = a(n).abs
    abs_fn(b)(n) = b(n).abs
}

/// Difference between two partial sums of the same series.
define partial_diff(f: Nat -> Real, m: Nat, n: Nat) -> Real {
    partial(f, n) - partial(f, m)
}

/// Difference between the limit of partial sums and the mth partial sum.
define tail_limit_diff(f: Nat -> Real, m: Nat) -> Real {
    limit(partial(f)) - partial(f, m)
}

/// Pointwise difference of two sequences.
define sub_seq(s: Nat -> Real, t: Nat -> Real, n: Nat) -> Real {
    s(n) - t(n)
}

/// Absolute value of the difference between two sequences.
define abs_diff_seq(s: Nat -> Real, t: Nat -> Real, n: Nat) -> Real {
    sub_seq(s, t, n).abs
}

/// Two sequences differ by a vanishing amount.
define vanishing_diff(s: Nat -> Real, t: Nat -> Real) -> Bool {
    vanishes(abs_diff_seq(s, t))
}

/// A sequence vanishes if and only if its absolute value sequence vanishes.
theorem vanishes_of_abs_seq(a: Nat -> Real) {
    vanishes(abs_seq(a))
    implies
    vanishes(a)
} by {
    seq_lte(abs_seq(a), abs_seq(abs_seq(a)))
}

/// Taking negatives preserves the vanishing property.
theorem vanishes_neg_seq(a: Nat -> Real) {
    vanishes(a)
    implies
    vanishes(neg_seq(a))
} by {
    not vanishes(a) or converges(a)
    not vanishes(a) or limit(a) = Real.0
    not converges(a) or converges(neg_seq(a))
    not converges(a) or converges_to(neg_seq(a), -limit(a))
    not converges(neg_seq(a)) or converges_to(neg_seq(a), limit(neg_seq(a)))
    not converges_to(neg_seq(a), Real.0) or not converges_to(neg_seq(a), limit(neg_seq(a))) or limit(neg_seq(a)) = Real.0
    limit(neg_seq(a)) != Real.0 or not converges(neg_seq(a)) or vanishes(neg_seq(a))
}

/// Vanishing is preserved under pointwise equality.
theorem vanishes_congr_left(a: Nat -> Real, b: Nat -> Real) {
    a = b and vanishes(b)
    implies
    vanishes(a)
}

/// Negating the difference sequence swaps the order of the arguments.
theorem neg_seq_sub_seq_eq(s: Nat -> Real, t: Nat -> Real) {
    neg_seq(sub_seq(s, t)) = sub_seq(t, s)
} by {
    forall(n: Nat) {
        let lhs = sub_seq(s, t, n)
        let rhs = sub_seq(t, s, n)

        lhs + rhs = (s(n) - t(n)) + (t(n) - s(n))
        s(n) + -t(n) + (t(n) + -s(n)) = s(n) + (-t(n) + (t(n) + -s(n)))
        -t(n) + t(n) + -s(n) = -t(n) + (t(n) + -s(n))
        -t(n) + -(-t(n)) = Real.0
        -(-t(n)) = t(n)
        s(n) + -s(n) = Real.0


        neg_seq(sub_seq(s, t))(n) = sub_seq(t, s, n)
    }
}

/// Absolute values of the raw difference equal the absolute difference sequence (original order).
theorem abs_seq_sub_seq_eq_abs_diff_left(s: Nat -> Real, t: Nat -> Real) {
    abs_seq(sub_seq(s, t)) = abs_diff_seq(s, t)
} by {
    forall(n: Nat) {
        sub_seq(s, t, n).abs = abs_diff_seq(s, t, n)
        sub_seq(s, t, n).abs = abs_seq(sub_seq(s, t), n)
        abs_seq(sub_seq(s, t), n) = abs_diff_seq(s, t, n)
    }
}

/// Absolute values of the reversed difference also equal the absolute difference sequence.
theorem abs_seq_sub_seq_eq_abs_diff_right(s: Nat -> Real, t: Nat -> Real) {
    abs_seq(sub_seq(t, s)) = abs_diff_seq(s, t)
} by {
    forall(n: Nat) {
        sub_seq(t, s, n) = -sub_seq(s, t, n)
        abs_seq(sub_seq(t, s))(n) = abs_diff_seq(s, t, n)
    }
}

/// Absolute vanishing differences imply raw differences vanish.
theorem vanishing_diff_abs_eq(s: Nat -> Real, t: Nat -> Real) {
    vanishing_diff(s, t)
    implies
    vanishes(sub_seq(t, s))
}

/// Vanishing differences allow us to transfer limits across sequences.
theorem limit_preserved_by_vanishing_diff(s: Nat -> Real, t: Nat -> Real) {
    converges(s) and vanishing_diff(s, t)
    implies
    converges(t) and limit(s) = limit(t)
} by {
    if converges(s) and vanishing_diff(s, t) {
        let diff = sub_seq(t, s)

        vanishing_diff(s, t) implies vanishes(sub_seq(t, s))

        forall(n: Nat) {
            t(n) + -s(n) + s(n) = t(n) + (-s(n) + s(n))
            s(n) + -s(n) = Real.0
            t(n) + Real.0 = t(n)
            add_seq(diff, s)(n) = t(n)
        }

        converges(diff) and converges(s) implies converges_to(add_seq(diff, s), limit(diff) + limit(s))
        Real.0 + limit(s) = limit(s)
        converges_to(t, limit(s))
    }
}

/// Absolute convergence implies convergence (alternate formulation).
/// This connects our definition to the existing theorem in real_series.ac.
theorem absolutely_converges_imp_converges(a: Nat -> Real) {
    absolutely_converges(a) implies converges(partial(a))
} by {
    from real.real_series import abs_conv_imp_conv

    if absolutely_converges(a) {
        partial(abs_fn(a)) = partial(compose(Real.abs, a))

        converges(partial(a))
    }
}

/// Absolute convergence is preserved under scalar multiplication.
theorem absolutely_converges_scalar_mul(c: Real, a: Nat -> Real) {
    absolutely_converges(a) implies absolutely_converges(mul_fn(c, a))
} by {
    from real.real_series import converges_mul_seq
    from list import partial_pointwise_eq, partial_scalar_mul

    if absolutely_converges(a) {
        forall(n: Nat) {
            forall(k: Nat) {
                if k < n {
                    mul_fn(c.abs, abs_fn(a), k) = c.abs * abs_fn(a)(k)
                    abs_fn(mul_fn(c, a))(k) = mul_fn(c.abs, abs_fn(a))(k)
                }
            }
            partial(abs_fn(mul_fn(c, a)), n) = partial(mul_fn(c.abs, abs_fn(a)), n)
        }
        partial(abs_fn(mul_fn(c, a))) = partial(mul_fn(c.abs, abs_fn(a)))

        forall(n: Nat) {
            mul_fn(c.abs, partial(abs_fn(a)), n) = c.abs * partial(abs_fn(a), n)
            partial(mul_fn(c.abs, abs_fn(a)), n) = mul_fn(c.abs, partial(abs_fn(a)), n)
        }
        partial(mul_fn(c.abs, abs_fn(a))) = mul_fn(c.abs, partial(abs_fn(a)))

        forall(n: Nat) {
            mul_seq(c.abs, partial(abs_fn(a)), n) = c.abs * partial(abs_fn(a), n)
            mul_fn(c.abs, partial(abs_fn(a)), n) = c.abs * partial(abs_fn(a), n)
            mul_fn(c.abs, partial(abs_fn(a)), n) = mul_seq(c.abs, partial(abs_fn(a)), n)
        }
        mul_fn(c.abs, partial(abs_fn(a))) = mul_seq(c.abs, partial(abs_fn(a)))

        converges(partial(abs_fn(a))) implies converges(mul_seq(c.abs, partial(abs_fn(a))))
        absolutely_converges(mul_fn(c, a))
    }
}

/// Absolute convergence is preserved by multiplication with a bounded sequence.
/// If |b(n)| ≤ bound for all n and ∑|a(n)| converges, then ∑|a(n) * b(n)| converges.
theorem absolutely_converges_mul_bounded(a: Nat -> Real, b: Nat -> Real) {
    absolutely_converges(a)
    and has_upper_bound_seq(abs_fn(b))
    implies
    absolutely_converges(prod_seq(a, b))
} by {
    from real.real_series import converges_mul_seq
    from list import partial_scalar_mul

    let bound: Real satisfy {
        is_upper_bound(abs_fn(b), bound)
    }

    is_upper_bound(abs_fn(b), bound)

    abs_fn(b)(Nat.0) >= Real.0
    abs_fn(b)(Nat.0) <= bound
    Real.0 <= bound

    forall(n: Nat) {
        is_upper_bound(abs_fn(b), bound)
        abs_fn(b)(n) <= bound
        abs_fn(a)(n) * abs_fn(b)(n) <= abs_fn(a)(n) * bound
        abs_fn(prod_seq(a, b))(n) = abs_fn(a)(n) * abs_fn(b)(n)
        abs_fn(prod_seq(a, b))(n) <= abs_fn(a)(n) * bound
        abs_fn(a)(n) * bound = bound * abs_fn(a)(n)
        mul_fn(bound, abs_fn(a))(n) = bound * abs_fn(a)(n)
        abs_fn(prod_seq(a, b))(n) <= mul_fn(bound, abs_fn(a))(n)
    }

    forall(n: Nat) {
        abs_fn(prod_seq(a, b))(n) >= Real.0
    }
    is_lower_bound(abs_fn(prod_seq(a, b)), Real.0)
    seq_lte(abs_fn(prod_seq(a, b)), mul_fn(bound, abs_fn(a)))

    forall(n: Nat) {
        mul_fn(bound, partial(abs_fn(a)), n) = bound * partial(abs_fn(a), n)
        partial(mul_fn(bound, abs_fn(a)), n) = mul_fn(bound, partial(abs_fn(a)), n)
    }
    partial(mul_fn(bound, abs_fn(a))) = mul_fn(bound, partial(abs_fn(a)))
    forall(n: Nat) {
        mul_seq(bound, partial(abs_fn(a)), n) = bound * partial(abs_fn(a), n)
        mul_fn(bound, partial(abs_fn(a)), n) = bound * partial(abs_fn(a), n)
        mul_fn(bound, partial(abs_fn(a)), n) = mul_seq(bound, partial(abs_fn(a)), n)
    }
    mul_fn(bound, partial(abs_fn(a))) = mul_seq(bound, partial(abs_fn(a)))

    absolutely_converges(a)
    converges(partial(abs_fn(a)))
    converges(mul_seq(bound, partial(abs_fn(a))))
        converges(partial(mul_fn(bound, abs_fn(a))))

        converges(partial(abs_fn(prod_seq(a, b))))
        absolutely_converges(prod_seq(a, b))
}

/// Absolute convergence is preserved under addition of sequences.
theorem absolutely_converges_add(a: Nat -> Real, b: Nat -> Real) {
    absolutely_converges(a) and absolutely_converges(b)
    implies
    absolutely_converges(add_fn(a, b))
} by {
    from real.real_series import add_seq_converges
    from list import partial_add

    if absolutely_converges(a) and absolutely_converges(b) {
        forall(n: Nat) {
            (a(n) + b(n)).abs <= a(n).abs + b(n).abs
            add_fn[Nat, Real](a, b, n) = a(n) + b(n)
            add_fn[Nat, Real](abs_fn(a), abs_fn(b), n) = abs_fn(a, n) + abs_fn(b, n)
            a(n).abs = abs_fn(a, n)
            add_fn[Nat, Real](a, b, n).abs = abs_fn(add_fn[Nat, Real](a, b), n)
            b(n).abs = abs_fn(b, n)
            abs_fn(add_fn(a, b))(n) <= add_fn(abs_fn(a), abs_fn(b))(n)
        }

        forall(n: Nat) {
            abs_fn(add_fn(a, b))(n) >= Real.0
        }
        is_lower_bound(abs_fn(add_fn(a, b)), Real.0)
        seq_lte(abs_fn(add_fn(a, b)), add_fn(abs_fn(a), abs_fn(b)))

        forall(n: Nat) {
            add_fn(partial(abs_fn(a)), partial(abs_fn(b)), n) = partial(abs_fn(a), n) + partial(abs_fn(b), n)
            partial(add_fn(abs_fn(a), abs_fn(b)), n) = add_fn(partial(abs_fn(a)), partial(abs_fn(b)), n)
        }
        partial(add_fn(abs_fn(a), abs_fn(b))) = add_fn(partial(abs_fn(a)), partial(abs_fn(b)))

        forall(n: Nat) {
            add_seq(partial(abs_fn(a)), partial(abs_fn(b)), n) = partial(abs_fn(a), n) + partial(abs_fn(b), n)
            add_fn(partial(abs_fn(a)), partial(abs_fn(b)), n) = partial(abs_fn(a), n) + partial(abs_fn(b), n)
            add_fn(partial(abs_fn(a)), partial(abs_fn(b)), n) = add_seq(partial(abs_fn(a)), partial(abs_fn(b)), n)
        }
        add_fn(partial(abs_fn(a)), partial(abs_fn(b))) = add_seq(partial(abs_fn(a)), partial(abs_fn(b)))

        converges(partial(abs_fn(a))) and converges(partial(abs_fn(b))) implies converges(add_seq(partial(abs_fn(a)), partial(abs_fn(b))))

        is_lower_bound(abs_fn(add_fn(a, b)), Real.0) and seq_lte(abs_fn(add_fn(a, b)), add_fn(abs_fn(a), abs_fn(b))) and converges(partial(add_fn(abs_fn(a), abs_fn(b)))) implies converges(partial(abs_fn(add_fn(a, b))))
        converges(partial(abs_fn(add_fn(a, b))))
        absolutely_converges(add_fn(a, b))
    }
}

/// Filtering a sequence preserves absolute convergence.
theorem absolutely_converges_filter_seq(a: Nat -> Real, p: Nat -> Bool) {
    absolutely_converges(a)
    implies
    absolutely_converges(filter_seq(a, p))
} by {
    if absolutely_converges(a) {
        forall(n: Nat) {
            if p(n) {
                filter_seq(a, p)(n) = if p(n) { a(n) } else { Real.0 }
                (if p(n) { a(n) } else { Real.0 }) = a(n)
                filter_seq(a, p)(n) = a(n)
                abs_fn(filter_seq(a, p))(n) = abs_fn(a)(n)
                abs_fn(filter_seq(a, p))(n) <= abs_fn(a)(n)
            } else {
                filter_seq(a, p)(n) = if p(n) { a(n) } else { Real.0 }
                (if p(n) { a(n) } else { Real.0 }) = Real.0
                filter_seq(a, p)(n) = Real.0
                abs_fn(filter_seq(a, p))(n) = Real.0
                Real.0 <= abs_fn(a)(n)
                abs_fn(filter_seq(a, p))(n) <= abs_fn(a)(n)
            }
        }

        forall(n: Nat) {
            abs_fn(filter_seq(a, p))(n) >= Real.0
        }
        is_lower_bound(abs_fn(filter_seq(a, p)), Real.0)
        seq_lte(abs_fn(filter_seq(a, p)), abs_fn(a))

        absolutely_converges(a)
        converges(partial(abs_fn(a)))

        converges(partial(abs_fn(filter_seq(a, p))))
        absolutely_converges(filter_seq(a, p))
    }
}

/// A mask of an absolutely convergent series is also absolutely convergent.
theorem absolutely_converges_mask(a: Nat -> Real, s: Set[Nat]) {
    absolutely_converges(a)
    implies
    absolutely_converges(mask(a, s))
} by {
    if absolutely_converges(a) {
        forall(n: Nat) {
            mask(a, s, n) = filter_seq(a, s.contains, n)
        }
        mask(a, s) = filter_seq(a, s.contains)

        absolutely_converges(mask(a, s))
    }
}

/// Comparison test for absolute convergence.
/// If |a(n)| ≤ b(n), the b(n) are nonnegative, and ∑ b(n) converges, then ∑ a(n) converges absolutely.
theorem absolutely_converges_comparison(a: Nat -> Real, b: Nat -> Real) {
    is_lower_bound(b, Real.0)
    and seq_lte(abs_fn(a), b)
    and converges(partial(b))
    implies
    absolutely_converges(a)
} by {
    forall(n: Nat) {
    }
    is_lower_bound(abs_fn(a), Real.0)

    converges(partial(abs_fn(a)))
    absolutely_converges(a)
}

/// Tail bound for absolutely convergent series.
/// If a series converges absolutely, then tail sums can be made arbitrarily small.
/// For any ε > 0, there exists N such that for all n, m ≥ N with n ≤ m,
/// the sum of |a(k)| from k=n to k=m-1 is less than ε.
/// For fixed m, the difference between partial sums converges to the tail limit difference.
theorem tail_diff_converges(a: Nat -> Real, m: Nat) {
    absolutely_converges(a)
    implies
    converges_to(partial_diff(abs_fn(a), m), tail_limit_diff(abs_fn(a), m))
} by {
    if absolutely_converges(a) {
        let lim = limit(partial(abs_fn(a)))


        converges(partial(abs_fn(a))) and converges(constant[Nat, Real](-partial(abs_fn(a), m))) implies converges_to(add_seq(partial(abs_fn(a)), constant[Nat, Real](-partial(abs_fn(a), m))), limit(partial(abs_fn(a))) + limit(constant[Nat, Real](-partial(abs_fn(a), m))))

        forall(n: Nat) {
            partial(abs_fn(a), n) - partial(abs_fn(a), m) = partial_diff(abs_fn(a), m, n)
            partial(abs_fn(a), n) + constant(-partial(abs_fn(a), m), n) = add_seq(partial(abs_fn(a)), constant[Nat, Real](-partial(abs_fn(a), m)), n)
            partial(abs_fn(a), n) + -partial(abs_fn(a), m) = partial(abs_fn(a), n) - partial(abs_fn(a), m)
            constant(-partial(abs_fn(a), m), n) = -partial(abs_fn(a), m)
            partial_diff(abs_fn(a), m, n) = add_seq(partial(abs_fn(a)), constant[Nat, Real](-partial(abs_fn(a), m)), n)
        }
        partial_diff(abs_fn(a), m) =
            add_seq(partial(abs_fn(a)), constant[Nat, Real](-partial(abs_fn(a), m)))

        converges(partial(abs_fn(a))) = absolutely_converges(a)
        limit(partial(abs_fn(a))) - partial(abs_fn(a), m) = tail_limit_diff(abs_fn(a), m)
        converges_to(partial_diff(abs_fn(a), m), tail_limit_diff(abs_fn(a), m))
    }
}

theorem abs_conv_tail_bound(a: Nat -> Real, eps: Real) {
    absolutely_converges(a) and eps.is_positive
    implies
    exists(big_n: Nat) {
        forall(n: Nat, m: Nat) {
            big_n <= n and n <= m
            implies
            partial(abs_fn(a), m) - partial(abs_fn(a), n) < eps
        }
    }
} by {
    from real.real_series import distant_increasing, nonneg_imp_partial_increasing

    if absolutely_converges(a) and eps.is_positive {
        let big_n: Nat satisfy {
            cauchy_bound(partial(abs_fn(a)), big_n, eps)
        }

        forall(n: Nat, m: Nat) {
            if big_n <= n and n <= m {


                big_n <= m
                partial(abs_fn(a), m).is_close(partial(abs_fn(a), n), eps)
                (partial(abs_fn(a), m) - partial(abs_fn(a), n)).abs < eps
                partial(abs_fn(a), m) - partial(abs_fn(a), n) <= (partial(abs_fn(a), m) - partial(abs_fn(a), n)).abs
                partial(abs_fn(a), m) - partial(abs_fn(a), n) < eps
            }
        }
    }
}
/// Absolute convergence ensures the infinite tail of absolute values is arbitrarily small.
/// Given ε > 0, there exists N such that the sum of |a(k)| for k ≥ N is less than ε.
theorem abs_conv_tail_sum_small(a: Nat -> Real, eps: Real) {
    absolutely_converges(a) and eps.is_positive
    implies
    exists(big_n: Nat) {
        tail_limit_diff(abs_fn(a), big_n) < eps
    }
} by {
    if absolutely_converges(a) and eps.is_positive {
        let eps_small: Real satisfy {
            eps_small.is_positive and eps_small + eps_small < eps
        }

        let big_n: Nat satisfy {
            forall(n: Nat, m: Nat) {
                big_n <= n and n <= m
                implies
                partial(abs_fn(a), m) - partial(abs_fn(a), n) < eps_small
            }
        }

        forall(n: Nat) {
            if big_n <= n {
                partial(abs_fn(a), n) - partial(abs_fn(a), big_n) < eps_small
                partial_diff(abs_fn(a), big_n, n) <= eps_small
            }
        }

        exists(n0: Nat) {
            forall(i: Nat) {
                n0 <= i implies partial_diff(abs_fn(a), big_n, i) <= eps_small
            }
        }
        eventual_ub(partial_diff(abs_fn(a), big_n), eps_small)

        converges(partial_diff(abs_fn(a), big_n))

        converges_to(partial_diff(abs_fn(a), big_n), limit(partial_diff(abs_fn(a), big_n)))
        tail_limit_diff(abs_fn(a), big_n) = limit(partial_diff(abs_fn(a), big_n))

        tail_limit_diff(abs_fn(a), big_n) <= eps_small

        eps_small < eps_small + eps_small

        tail_limit_diff(abs_fn(a), big_n) < eps
    }
}

/// The absolute value function commutes with the tail operation.
/// For any sequence a and offset n, abs_fn(tail(a, n)) = tail(abs_fn(a), n).
theorem abs_fn_tail_comm(a: Nat -> Real, n: Nat) {
    abs_fn(tail(a, n)) = tail(abs_fn(a), n)
} by {
    forall(i: Nat) {
        abs_fn(tail(a, n))(i) = tail(a, n, i).abs
        tail(a, n, i) = a(n + i)
        tail(a, n, i).abs = a(n + i).abs
        abs_fn(a)(n + i) = a(n + i).abs
        tail(abs_fn(a), n)(i) = abs_fn(a)(n + i)
        abs_fn(tail(a, n))(i) = tail(abs_fn(a), n)(i)
    }
}

/// If the tail of a series converges absolutely, then the series itself converges absolutely.
/// This is a key property showing that absolute convergence depends only on the tail behavior.
theorem abs_conv_tail_imp_abs_conv(a: Nat -> Real, n: Nat) {
    absolutely_converges(tail(a, n)) implies absolutely_converges(a)
} by {
    from real.real_series import partial_tail_conv_imp_partial_conv

    if absolutely_converges(tail(a, n)) {
        converges(partial(abs_fn(tail(a, n))))
        abs_fn(tail(a, n)) = tail(abs_fn(a), n)
        converges(partial(tail(abs_fn(a), n)))
        converges(partial(abs_fn(a)))
        absolutely_converges(a)
    }
}

/// Pointwise addition of two mask with disjoint index sets equals the mask of their union.
theorem mask_disjoint_add(f: Nat -> Real, s1: Set[Nat], s2: Set[Nat], u: Set[Nat]) {
    s1.is_disjoint(s2) and u = s1.union(s2)
    implies
    add_fn(mask(f, s1), mask(f, s2)) = mask(f, u)
} by {
    if s1.is_disjoint(s2) and u = s1.union(s2) {
        forall(n: Nat) {
            if s1.contains(n) {
                mask(f, s1, n) = f(n)
                not s2.contains(n)
                mask(f, s2, n) = Real.0
                u.contains(n)
                mask(f, u, n) = f(n)
                add_fn(mask(f, s1), mask(f, s2))(n) = mask(f, s1, n) + mask(f, s2, n)
                mask(f, s1, n) + mask(f, s2, n) = f(n) + Real.0
                add_fn(mask(f, s1), mask(f, s2))(n) = mask(f, u, n)
            } else {
                mask(f, s1, n) = Real.0
                if s2.contains(n) {
                    mask(f, s2, n) = f(n)
                    u.contains(n)
                    mask(f, u, n) = f(n)
                    add_fn(mask(f, s1), mask(f, s2))(n) = mask(f, s1, n) + mask(f, s2, n)
                    mask(f, s1, n) + mask(f, s2, n) = Real.0 + f(n)
                    add_fn(mask(f, s1), mask(f, s2))(n) = mask(f, u, n)
                } else {
                    mask(f, s2, n) = Real.0
                    not u.contains(n)
                    mask(f, u, n) = Real.0
                    add_fn(mask(f, s1), mask(f, s2))(n) = mask(f, s1, n) + mask(f, s2, n)
                    mask(f, s1, n) + mask(f, s2, n) = Real.0 + Real.0
                    add_fn(mask(f, s1), mask(f, s2))(n) = mask(f, u, n)
                }
            }
        }
    }
}

/// For an absolutely convergent series and disjoint sets s1 and s2 with union u,
/// the sum of the mask for u equals the sum of the mask for s1 plus s2.
theorem mask_disjoint_union_sum(f: Nat -> Real, s1: Set[Nat], s2: Set[Nat], u: Set[Nat]) {
    absolutely_converges(f) and s1.is_disjoint(s2) and u = s1.union(s2)
    implies
    limit(partial(mask(f, u))) = limit(partial(mask(f, s1))) + limit(partial(mask(f, s2)))
} by {
    from real.real_series import add_seq_converges
    from list import partial_add

    if absolutely_converges(f) and s1.is_disjoint(s2) and u = s1.union(s2) {


        converges(partial(mask(f, s1)))
        converges(partial(mask(f, s2)))
        converges(partial(mask(f, u)))

        add_fn(mask(f, s1), mask(f, s2)) = mask(f, u)

        partial(add_fn(mask(f, s1), mask(f, s2))) = partial(mask(f, u))
        forall(n: Nat) {
            add_fn(partial(mask(f, s1)), partial(mask(f, s2)), n) = partial(mask(f, s1), n) + partial(mask(f, s2), n)
            partial(add_fn(mask(f, s1), mask(f, s2)), n) = add_fn(partial(mask(f, s1)), partial(mask(f, s2)), n)
        }
        partial(add_fn(mask(f, s1), mask(f, s2))) = add_fn(partial(mask(f, s1)), partial(mask(f, s2)))
        forall(n: Nat) {
            add_seq(partial(mask(f, s1)), partial(mask(f, s2)), n) = partial(mask(f, s1), n) + partial(mask(f, s2), n)
            add_fn(partial(mask(f, s1)), partial(mask(f, s2)), n) = partial(mask(f, s1), n) + partial(mask(f, s2), n)
            add_fn(partial(mask(f, s1)), partial(mask(f, s2)), n) = add_seq(partial(mask(f, s1)), partial(mask(f, s2)), n)
        }
        add_fn(partial(mask(f, s1)), partial(mask(f, s2))) = add_seq(partial(mask(f, s1)), partial(mask(f, s2)))

        converges(partial(mask(f, s1))) and converges(partial(mask(f, s2))) implies converges_to(add_seq(partial(mask(f, s1)), partial(mask(f, s2))), limit(partial(mask(f, s1))) + limit(partial(mask(f, s2))))
        limit(add_seq(partial(mask(f, s1)), partial(mask(f, s2)))) = limit(partial(mask(f, s1))) + limit(partial(mask(f, s2)))

        limit(partial(mask(f, u))) = limit(partial(mask(f, s1))) + limit(partial(mask(f, s2)))
    }
}

/// Absolute value commutes with mask.
theorem abs_fn_mask(f: Nat -> Real, s: Set[Nat]) {
    abs_fn(mask(f, s)) = mask(abs_fn(f), s)
} by {
    forall(n: Nat) {
        abs_fn(mask(f, s))(n) = mask(f, s, n).abs

        if s.contains(n) {
            mask(f, s, n) = f(n)
            mask(f, s, n).abs = f(n).abs
            mask(abs_fn(f), s, n) = abs_fn(f)(n)
            abs_fn(f)(n) = f(n).abs
            abs_fn(mask(f, s))(n) = mask(abs_fn(f), s, n)
        } else {
            mask(f, s, n) = Real.0
            mask(f, s, n).abs = Real.0
            mask(abs_fn(f), s, n) = Real.0
            abs_fn(mask(f, s))(n) = mask(abs_fn(f), s, n)
        }
    }
}

/// If s1 is a superset of s2, then the absolute mask sum for s1 is at least that for s2.
theorem mask_abs_sum_superset(f: Nat -> Real, s1: Set[Nat], s2: Set[Nat]) {
    absolutely_converges(f) and s1.superset(s2)
    implies
    limit(partial(mask(abs_fn(f), s1))) >= limit(partial(mask(abs_fn(f), s2)))
} by {
from data.basic.set import union_with_difference_decomp_rev
    from real.real_series import nonneg_partial_nonneg, partial_nonneg
    from list import partial_add

    if absolutely_converges(f) and s1.superset(s2) {
        let diff = s1.difference(s2)

        // Show s2 and diff are disjoint
        forall(x: Nat) {
            if s2.contains(x) and diff.contains(x) {
                not s2.contains(x)
            }
        }
        s2.is_disjoint(diff)

        // Show s1 = s2 ∪ diff
        s2.union(s1) = s2.union(s1.difference(s2))
        let u = s2.union(diff)

        forall(x: Nat) {
            if s1.contains(x) {
                u.contains(x)
            }
            if u.contains(x) {
                s1.contains(x)
            }
        }
        s1 = u

        // Subseries of f are absolutely convergent

        // Convert to partial sums of abs values

        forall(n: Nat) {
            partial(abs_fn(mask(f, s1)), n) = partial(mask(abs_fn(f), s1), n)
        }
        partial(abs_fn(mask(f, s1))) = partial(mask(abs_fn(f), s1))
        forall(n: Nat) {
            partial(abs_fn(mask(f, s2)), n) = partial(mask(abs_fn(f), s2), n)
        }
        partial(abs_fn(mask(f, s2))) = partial(mask(abs_fn(f), s2))
        forall(n: Nat) {
            partial(abs_fn(mask(f, diff)), n) = partial(mask(abs_fn(f), diff), n)
        }
        partial(abs_fn(mask(f, diff))) = partial(mask(abs_fn(f), diff))

        converges(partial(abs_fn(mask(f, s1))))
        converges(partial(abs_fn(mask(f, s2))))
        converges(partial(abs_fn(mask(f, diff))))

        converges(partial(mask(abs_fn(f), s1)))
        converges(partial(mask(abs_fn(f), s2)))
        converges(partial(mask(abs_fn(f), diff)))

        // Apply disjoint union theorem to abs_fn(f)
        // We prove this directly since abs_fn(f) is not "absolutely convergent"
        add_fn(mask(abs_fn(f), s2), mask(abs_fn(f), diff)) = mask(abs_fn(f), s1)

        partial(add_fn(mask(abs_fn(f), s2), mask(abs_fn(f), diff))) = partial(mask(abs_fn(f), s1))
        forall(n: Nat) {
            add_fn(partial(mask(abs_fn(f), s2)), partial(mask(abs_fn(f), diff)), n) = partial(mask(abs_fn(f), s2), n) + partial(mask(abs_fn(f), diff), n)
            partial(add_fn(mask(abs_fn(f), s2), mask(abs_fn(f), diff)), n) = add_fn(partial(mask(abs_fn(f), s2)), partial(mask(abs_fn(f), diff)), n)
        }
        partial(add_fn(mask(abs_fn(f), s2), mask(abs_fn(f), diff))) =
            add_fn(partial(mask(abs_fn(f), s2)), partial(mask(abs_fn(f), diff)))
        forall(n: Nat) {
            add_seq(partial(mask(abs_fn(f), s2)), partial(mask(abs_fn(f), diff)), n) = partial(mask(abs_fn(f), s2), n) + partial(mask(abs_fn(f), diff), n)
            add_fn(partial(mask(abs_fn(f), s2)), partial(mask(abs_fn(f), diff)), n) = partial(mask(abs_fn(f), s2), n) + partial(mask(abs_fn(f), diff), n)
            add_fn(partial(mask(abs_fn(f), s2)), partial(mask(abs_fn(f), diff)), n) = add_seq(partial(mask(abs_fn(f), s2)), partial(mask(abs_fn(f), diff)), n)
        }
        add_fn(partial(mask(abs_fn(f), s2)), partial(mask(abs_fn(f), diff))) =
            add_seq(partial(mask(abs_fn(f), s2)), partial(mask(abs_fn(f), diff)))

        converges(partial(mask(abs_fn(f), s2))) and converges(partial(mask(abs_fn(f), diff))) implies converges_to(add_seq(partial(mask(abs_fn(f), s2)), partial(mask(abs_fn(f), diff))), limit(partial(mask(abs_fn(f), s2))) + limit(partial(mask(abs_fn(f), diff))))
        limit(add_seq(partial(mask(abs_fn(f), s2)), partial(mask(abs_fn(f), diff)))) =
            limit(partial(mask(abs_fn(f), s2))) + limit(partial(mask(abs_fn(f), diff)))

        limit(partial(mask(abs_fn(f), s1))) =
            limit(partial(mask(abs_fn(f), s2))) + limit(partial(mask(abs_fn(f), diff)))

        // Show the diff sum is nonnegative
        forall(n: Nat) {

            if diff.contains(n) {
                mask(abs_fn(f), diff, n) = abs_fn(f)(n)
                mask(abs_fn(f), diff, n) >= Real.0
            } else {
                mask(abs_fn(f), diff, n) = Real.0
                mask(abs_fn(f), diff, n) >= Real.0
            }
            Real.0 <= mask(abs_fn(f), diff, n)
        }
        is_lower_bound(mask(abs_fn(f), diff), Real.0)

        forall(n: Nat) {
            partial(mask(abs_fn(f), diff), n) >= Real.0
        }
        let start = Nat.0
        forall(i: Nat) {
            if start <= i {
                partial(mask(abs_fn(f), diff), i) >= Real.0
                Real.0 <= partial(mask(abs_fn(f), diff), i)
            }
        }
        exists(n0: Nat) {
            forall(i: Nat) {
                n0 <= i implies Real.0 <= partial(mask(abs_fn(f), diff), i)
            }
        }
        eventual_lb(partial(mask(abs_fn(f), diff)), Real.0)

        converges(partial(mask(abs_fn(f), diff))) and eventual_lb(partial(mask(abs_fn(f), diff)), Real.0) implies Real.0 <= limit(partial(mask(abs_fn(f), diff)))
        limit(partial(mask(abs_fn(f), diff))) >= Real.0

        // From a = b + c and c >= 0, we get a >= b
        let c: Real = limit(partial(mask(abs_fn(f), diff)))
        let b: Real = limit(partial(mask(abs_fn(f), s2)))
        let a: Real = limit(partial(mask(abs_fn(f), s1)))
        a = b + c
        c >= Real.0
        b + c >= b + Real.0
        b + Real.0 = b
        a >= b
        limit(partial(mask(abs_fn(f), s1))) >= limit(partial(mask(abs_fn(f), s2)))
    }
}

/// If s1 is a subset of s2, then the absolute mask sum for s1 is at most that for s2.
theorem mask_abs_sum_subset(f: Nat -> Real, s1: Set[Nat], s2: Set[Nat]) {
    absolutely_converges(f) and s1.subset(s2)
    implies
    limit(partial(mask(abs_fn(f), s1))) <= limit(partial(mask(abs_fn(f), s2)))
} by {
    if absolutely_converges(f) and s1.subset(s2) {
        s2.superset(s1)
        limit(partial(mask(abs_fn(f), s2))) >= limit(partial(mask(abs_fn(f), s1)))
    }
}

/// For m >= n, the partial sum of the mask from n equals the tail partial sum.
theorem partial_mask_from_eq(a: Nat -> Real, n: Nat, m: Nat) {
    n <= m
    implies
    partial(mask(a, nat_from(n)), m) = partial(a, m) - partial(a, n)
} by {
    from real.real_series import partial_tail_sub, partial_tail, partial_all_zeros

    if n <= m {
        let d: Nat satisfy {
            n + d = m
        }

        define p(k: Nat) -> Bool {
            partial(mask(a, nat_from(n)), n + k) = partial(a, n + k) - partial(a, n)
        }

        // Base case: k = 0
        forall(i: Nat) {
            if i < n {
                not nat_from(n).contains(i)
                mask(a, nat_from(n), i) = Real.0
            }
        }
        partial(mask(a, nat_from(n)), n) = Real.0
        partial(a, n) - partial(a, n) = Real.0
        partial(mask(a, nat_from(n)), n) = partial(a, n) - partial(a, n)
        p(Nat.0)

        // Inductive step
        forall(k: Nat) {
            if p(k) {
                // Show p(k.suc)
                partial(mask(a, nat_from(n)), n + k.suc) =
                    partial(mask(a, nat_from(n)), n + k) + mask(a, nat_from(n), n + k)

                n + k >= n
                at_least(n, n + k)
                nat_from(n).contains(n + k)
                mask(a, nat_from(n), n + k) = a(n + k)

                partial(mask(a, nat_from(n)), n + k) = partial(a, n + k) - partial(a, n)

                partial(mask(a, nat_from(n)), n + k.suc) =
                    partial(a, n + k) - partial(a, n) + a(n + k)

                partial(a, n + k.suc) = partial(a, n + k) + a(n + k)

                partial(mask(a, nat_from(n)), n + k.suc) = partial(a, n + k.suc) - partial(a, n)
                p(k.suc)
            }
        }

        p(d)
        partial(mask(a, nat_from(n)), n + d) = partial(a, n + d) - partial(a, n)
        partial(mask(a, nat_from(n)), m) = partial(a, m) - partial(a, n)
    }
}

/// The limit of partial sums of the mask from n equals the tail limit.
theorem mask_from_eq_tail(a: Nat -> Real, n: Nat) {
    absolutely_converges(a)
    implies
    limit(partial(mask(abs_fn(a), nat_from(n)))) = tail_limit_diff(abs_fn(a), n)
} by {
    if absolutely_converges(a) {
        absolutely_converges(mask(a, nat_from(n)))
        abs_fn(mask(a, nat_from(n))) = mask(abs_fn(a), nat_from(n))

        partial(abs_fn(mask(a, nat_from(n)))) = partial(mask(abs_fn(a), nat_from(n)))

        converges(partial(abs_fn(mask(a, nat_from(n)))))
        converges(partial(mask(abs_fn(a), nat_from(n))))

        converges_to(partial_diff(abs_fn(a), n), tail_limit_diff(abs_fn(a), n))
        converges(partial_diff(abs_fn(a), n))

        forall(m: Nat) {
            if n <= m {
                partial(mask(abs_fn(a), nat_from(n)), m) = partial(abs_fn(a), m) - partial(abs_fn(a), n)
                partial_diff(abs_fn(a), n, m) = partial(abs_fn(a), m) - partial(abs_fn(a), n)
                partial(mask(abs_fn(a), nat_from(n)), m) = partial_diff(abs_fn(a), n, m)
            }
        }

        forall(m: Nat) {
            if n <= m {
                partial(mask(abs_fn(a), nat_from(n)), m) = partial_diff(abs_fn(a), n, m)
                partial(mask(abs_fn(a), nat_from(n)), m) - partial_diff(abs_fn(a), n, m) = Real.0
                sub_seq(partial(mask(abs_fn(a), nat_from(n))), partial_diff(abs_fn(a), n), m) = Real.0
                abs_diff_seq(partial(mask(abs_fn(a), nat_from(n))), partial_diff(abs_fn(a), n), m) = Real.0
            }
        }

        let diff_seq = abs_diff_seq(partial(mask(abs_fn(a), nat_from(n))), partial_diff(abs_fn(a), n))

        forall(eps: Real) {
            if eps.is_positive {
                forall(m: Nat) {
                    if n <= m {
                        diff_seq(m) = Real.0
                        diff_seq(m).is_close(Real.0, eps)
                    }
                }
                tail_bound(diff_seq, Real.0, n, eps)
                let tail_n: Nat satisfy {
                    tail_bound(diff_seq, Real.0, tail_n, eps)
                }
            }
        }

        converges_to(diff_seq, Real.0)
        vanishes(diff_seq)
        vanishing_diff(partial(mask(abs_fn(a), nat_from(n))), partial_diff(abs_fn(a), n))

        converges(partial(mask(abs_fn(a), nat_from(n)))) and
            limit(partial_diff(abs_fn(a), n)) = limit(partial(mask(abs_fn(a), nat_from(n))))

        converges_to(partial_diff(abs_fn(a), n), limit(partial_diff(abs_fn(a), n)))
        tail_limit_diff(abs_fn(a), n) = limit(partial_diff(abs_fn(a), n))

        limit(partial(mask(abs_fn(a), nat_from(n)))) = tail_limit_diff(abs_fn(a), n)
    }
}

/// For an absolutely convergent series, if a set has a lower bound of n,
/// the mask sum is at most the tail sum starting from n.
theorem mask_bounded_by_tail(a: Nat -> Real, s: Set[Nat], n: Nat) {
    absolutely_converges(a) and set_lower_bound(s, n)
    implies
    limit(partial(mask(abs_fn(a), s))) <= tail_limit_diff(abs_fn(a), n)
} by {
    if absolutely_converges(a) and set_lower_bound(s, n) {
        nat_from(n).superset(s)

        limit(partial(mask(abs_fn(a), s))) <= limit(partial(mask(abs_fn(a), nat_from(n))))

        limit(partial(mask(abs_fn(a), nat_from(n)))) = tail_limit_diff(abs_fn(a), n)

        limit(partial(mask(abs_fn(a), s))) <= tail_limit_diff(abs_fn(a), n)
    }
}

/// Helper: map(map(range, f), g) = map(range, compose(g, f))
theorem map_compose_helper(f: Nat -> Real, n: Nat) {
    map(map(n.range, f), Real.abs) = map(n.range, compose(Real.abs, f))
} by {
    forall(i: Nat) {
        if i < n.range.length {
            map(map(n.range, f), Real.abs) = map(n.range, compose(Real.abs, f))
        }
    }
}

/// Helper: partial of abs_fn equals sum of absolute values
theorem partial_abs_fn_eq(a: Nat -> Real, n: Nat) {
    partial(abs_fn(a), n) = sum(map(map(n.range, a), Real.abs))
} by {
    let items = map(n.range, a)
    partial(abs_fn(a), n) = sum(map(n.range, abs_fn(a)))

    forall(i: Nat) {
        abs_fn(a)(i) = a(i).abs
        compose(Real.abs, a)(i) = a(i).abs
        abs_fn(a)(i) = compose(Real.abs, a)(i)
    }
    abs_fn(a) = compose(Real.abs, a)
    map(n.range, abs_fn(a)) = map(n.range, compose(Real.abs, a))

    map(n.range, compose(Real.abs, a)) = map(map(n.range, a), Real.abs)
    map(items, Real.abs) = map(n.range, abs_fn(a))

    sum(map(n.range, abs_fn(a))) = sum(map(items, Real.abs))
    partial(abs_fn(a), n) = sum(map(items, Real.abs))
}

/// The absolute value of the sum of an absolutely convergent series
/// is bounded by the sum of the absolute values.
theorem abs_limit_le_abs_series(a: Nat -> Real) {
    absolutely_converges(a)
    implies
    limit(partial(a)).abs <= limit(partial(abs_fn(a)))
} by {
    from real.real_series import sum_abs_le_abs_sum, nonneg_partial_nonneg
    from real.limits import limit_abs_seq

    if absolutely_converges(a) {
        converges(partial(a))

        // Use triangle inequality on partial sums
        forall(n: Nat) {
            let items = map(n.range, a)
            partial(a, n) = sum(items)
            partial(a, n).abs = sum(items).abs

            sum(items).abs <= sum(map(items, Real.abs))

            partial(abs_fn(a), n) = sum(map(items, Real.abs))

            partial(a, n).abs <= partial(abs_fn(a), n)
        }

        // Take limit of both sides using limit_abs_seq
        converges_to(abs_seq(partial(a)), limit(partial(a)).abs)
        converges(abs_seq(partial(a)))

        forall(n: Nat) {
            abs_seq(partial(a))(n) = partial(a, n).abs
        }

        limit(abs_seq(partial(a))) = limit(partial(a)).abs

        // Now show limit(partial(a)).abs <= limit(partial(abs_fn(a)))
        // We have partial(a, n).abs <= partial(abs_fn(a), n) for all n
        // Both sequences converge, so limits preserve the inequality

        // Show that abs_seq(partial(a)) has limit(partial(abs_fn(a))) as an upper bound
        // partial(abs_fn(a)) is increasing and converges, so bounded by its limit
        from real.real_series import nonneg_partial_increasing, increasing_convergent_bounded_by_limit

        forall(n: Nat) {
        }
        is_lower_bound(abs_fn(a), Real.0)
        nonneg_partial_increasing(abs_fn(a))
        converges(partial(abs_fn(a)))
        is_upper_bound(partial(abs_fn(a)), limit(partial(abs_fn(a))))

        // So for all n, partial(abs_fn(a), n) <= limit(partial(abs_fn(a)))
        forall(n: Nat) {
            partial(abs_fn(a), n) <= limit(partial(abs_fn(a)))
        }

        // And abs_seq(partial(a))(n) <= partial(abs_fn(a), n) <= limit(partial(abs_fn(a)))
        forall(n: Nat) {
            abs_seq(partial(a))(n) <= partial(abs_fn(a), n)
            partial(abs_fn(a), n) <= limit(partial(abs_fn(a)))
            abs_seq(partial(a))(n) <= limit(partial(abs_fn(a)))
        }

        // So limit(partial(abs_fn(a))) is an upper bound for abs_seq(partial(a))
        is_upper_bound(abs_seq(partial(a)), limit(partial(abs_fn(a))))
        let start = Nat.0
        forall(i: Nat) {
            if start <= i {
                abs_seq(partial(a))(i) <= limit(partial(abs_fn(a)))
            }
        }
        exists(n0: Nat) {
            forall(i: Nat) {
                n0 <= i implies abs_seq(partial(a))(i) <= limit(partial(abs_fn(a)))
            }
        }
        eventual_ub(abs_seq(partial(a)), limit(partial(abs_fn(a))))

        limit(abs_seq(partial(a))) <= limit(partial(abs_fn(a)))
        limit(partial(a)).abs <= limit(partial(abs_fn(a)))
    }
}

/// For a decreasing sequence of sets with empty intersection,
/// the sequence of mask sums vanishes.
/// This captures the intuition that as sets shrink while avoiding all finite elements,
/// only arbitrarily large elements remain, making the mask sum arbitrarily small.
theorem decreasing_empty_intersection_mask_vanishes(a: Nat -> Real, s: Nat -> Set[Nat]) {
    absolutely_converges(a)
    and is_decreasing(s)
    and seq_intersection(s).is_empty
    implies
    vanishes(mask_seq_sum(a, s))
} by {
    if absolutely_converges(a) and is_decreasing(s) and seq_intersection(s).is_empty {
        // First show that mask_seq_sum(a, s) converges
        forall(i: Nat) {
            converges(partial(mask(a, s(i))))

            mask_seq_sum(a, s, i) = limit(partial(mask(a, s(i))))
        }

        // Now show it converges to zero
        forall(eps: Real) {
            if eps.is_positive {
                // Find tail_n such that tail sum is small
                let tail_n: Nat satisfy {
                    tail_limit_diff(abs_fn(a), tail_n) < eps
                }

                // Find big_n such that s(big_n) is bounded below by tail_n
                let big_n: Nat satisfy {
                    set_lower_bound(s(big_n), tail_n)
                }
                decreasing_seq_eventual_lower_bound(s, big_n, tail_n)

                forall(i: Nat) {
                    if i >= big_n {
                        // Later sets in the decreasing family inherit the tail lower bound.
                        set_lower_bound(s(i), tail_n)

                        // Apply mask_bounded_by_tail
                        limit(partial(mask(abs_fn(a), s(i)))) <= tail_limit_diff(abs_fn(a), tail_n)

                        // Use abs_limit_le_abs_series
                        limit(partial(mask(a, s(i)))).abs <= limit(partial(abs_fn(mask(a, s(i)))))

                        abs_fn(mask(a, s(i))) = mask(abs_fn(a), s(i))
                        partial(abs_fn(mask(a, s(i)))) = partial(mask(abs_fn(a), s(i)))
                        limit(partial(abs_fn(mask(a, s(i))))) = limit(partial(mask(abs_fn(a), s(i))))
                        limit(partial(mask(a, s(i)))).abs <= limit(partial(mask(abs_fn(a), s(i))))

                        mask_seq_sum(a, s, i) = limit(partial(mask(a, s(i))))
                        mask_seq_sum(a, s, i).abs <= limit(partial(mask(abs_fn(a), s(i))))

                        mask_seq_sum(a, s, i).abs <= tail_limit_diff(abs_fn(a), tail_n)

                        mask_seq_sum(a, s, i).abs < eps

                        mask_seq_sum(a, s, i).is_close(Real.0, eps)
                    }
                }
                forall(i: Nat) {
                    big_n <= i implies mask_seq_sum(a, s, i).is_close(Real.0, eps)
                }
                tail_bound(mask_seq_sum(a, s), Real.0, big_n, eps)
                let tail_n_witness: Nat satisfy {
                    tail_bound(mask_seq_sum(a, s), Real.0, tail_n_witness, eps)
                }
            }
        }
        converges_to(mask_seq_sum(a, s), Real.0)
        converges(mask_seq_sum(a, s))
        limit(mask_seq_sum(a, s)) = Real.0
        vanishes(mask_seq_sum(a, s))
    }
}

/// For an absolutely convergent series and an increasing sequence of sets whose union is universal,
/// the sequence of mask sums converges to the limit of the series.
theorem increasing_universal_union_mask_limit(a: Nat -> Real, s: Nat -> Set[Nat]) {
    absolutely_converges(a)
    and is_increasing(s)
    and seq_union(s) = Set[Nat].universal_set
    implies
    converges_to(mask_seq_sum(a, s), limit(partial(a)))
} by {
    if absolutely_converges(a) and is_increasing(s) and seq_union(s) = Set[Nat].universal_set {
        let sc = seq_complement(s)

        // The complement sequence is decreasing with empty intersection
        seq_complement_is_decreasing_of_increasing(s)
        is_decreasing(sc)

        seq_intersection(sc).is_empty

        // So the mask sums for complements vanish
        vanishes(mask_seq_sum(a, sc))

        // Now show that mask_seq_sum(a, s) + mask_seq_sum(a, sc) = limit(partial(a))
        forall(i: Nat) {
            // sc(i) is the complement of s(i)
            sc(i) = s(i).c

            // s(i) and its complement are disjoint
            set_is_disjoint_compl(s(i))
            s(i).is_disjoint(sc(i))

            // s(i) ∪ s(i).c = universal
            union_compl_is_universal(s(i))
            s(i).union(s(i).c) = Set[Nat].universal_set
            s(i).union(sc(i)) = Set[Nat].universal_set

            // For any function f, mask(f, universal) = f
            forall(n: Nat) {
                Set[Nat].universal_set.contains(n)
                mask(a, Set[Nat].universal_set, n) = a(n)
            }
            mask(a, Set[Nat].universal_set) = a

            mask(a, s(i).union(sc(i))) = a

            // Apply disjoint union theorem
            limit(partial(mask(a, s(i).union(sc(i))))) =
                limit(partial(mask(a, s(i)))) + limit(partial(mask(a, sc(i))))

            partial(mask(a, s(i).union(sc(i)))) = partial(a)
            limit(partial(a)) =
                limit(partial(mask(a, s(i)))) + limit(partial(mask(a, sc(i))))

            mask_seq_sum(a, s, i) = limit(partial(mask(a, s(i))))
            mask_seq_sum(a, sc, i) = limit(partial(mask(a, sc(i))))

            mask_seq_sum(a, s, i) + mask_seq_sum(a, sc, i) = limit(partial(a))
        }

        // mask_seq_sum(a, s) = limit(partial(a)) - mask_seq_sum(a, sc)
        forall(i: Nat) {
            mask_seq_sum(a, s, i) = limit(partial(a)) - mask_seq_sum(a, sc, i)
            sub_seq(mask_seq_sum(a, s), constant[Nat, Real](limit(partial(a))), i) =
                -mask_seq_sum(a, sc, i)
        }

        forall(i: Nat) {
            sub_seq(mask_seq_sum(a, s), constant[Nat, Real](limit(partial(a))), i) =
                -mask_seq_sum(a, sc, i)
            mul_seq(-1, mask_seq_sum(a, sc), i) = neg_seq(mask_seq_sum(a, sc), i)
            -mask_seq_sum(a, sc, i) = neg_seq(mask_seq_sum(a, sc), i)
            sub_seq(mask_seq_sum(a, s), constant[Nat, Real](limit(partial(a))), i) =
                neg_seq(mask_seq_sum(a, sc), i)
        }
        sub_seq(mask_seq_sum(a, s), constant[Nat, Real](limit(partial(a)))) =
            neg_seq(mask_seq_sum(a, sc))

        // Since mask_seq_sum(a, sc) vanishes, so does its negation
        vanishes(neg_seq(mask_seq_sum(a, sc)))

        forall(i: Nat) {
            abs_diff_seq(mask_seq_sum(a, s), constant[Nat, Real](limit(partial(a))), i) =
                sub_seq(mask_seq_sum(a, s), constant[Nat, Real](limit(partial(a))), i).abs
        }

        forall(i: Nat) {
            abs_seq(neg_seq(mask_seq_sum(a, sc)))(i) =
                neg_seq(mask_seq_sum(a, sc), i).abs
            neg_seq(mask_seq_sum(a, sc), i) = -mask_seq_sum(a, sc, i)
            neg_seq(mask_seq_sum(a, sc), i).abs = mask_seq_sum(a, sc, i).abs
            abs_seq(mask_seq_sum(a, sc))(i) = mask_seq_sum(a, sc, i).abs
            abs_seq(neg_seq(mask_seq_sum(a, sc)))(i) = abs_seq(mask_seq_sum(a, sc))(i)
        }
        abs_seq(neg_seq(mask_seq_sum(a, sc))) = abs_seq(mask_seq_sum(a, sc))

        vanishes(mask_seq_sum(a, sc))
        vanishes(abs_seq(mask_seq_sum(a, sc)))

        vanishes(abs_seq(neg_seq(mask_seq_sum(a, sc))))


        abs_seq(sub_seq(constant[Nat, Real](limit(partial(a))), mask_seq_sum(a, s))) =
            abs_diff_seq(mask_seq_sum(a, s), constant[Nat, Real](limit(partial(a))))

        forall(i: Nat) {
            sub_seq(constant[Nat, Real](limit(partial(a))), mask_seq_sum(a, s), i) =
                limit(partial(a)) - mask_seq_sum(a, s, i)
            limit(partial(a)) - mask_seq_sum(a, s, i) = mask_seq_sum(a, sc, i)
            sub_seq(constant[Nat, Real](limit(partial(a))), mask_seq_sum(a, s), i) =
                mask_seq_sum(a, sc, i)
        }
        sub_seq(constant[Nat, Real](limit(partial(a))), mask_seq_sum(a, s)) =
            mask_seq_sum(a, sc)

        abs_seq(mask_seq_sum(a, sc)) =
            abs_diff_seq(mask_seq_sum(a, s), constant[Nat, Real](limit(partial(a))))

        vanishing_diff(mask_seq_sum(a, s), constant[Nat, Real](limit(partial(a))))

        // Now show mask_seq_sum(a, s) converges
        forall(i: Nat) {
            converges(partial(mask(a, s(i))))
        }

        converges(constant[Nat, Real](limit(partial(a))))

        // Swap the order to match limit_preserved_by_vanishing_diff
        abs_seq(sub_seq(constant[Nat, Real](limit(partial(a))), mask_seq_sum(a, s))) =
            abs_diff_seq(constant[Nat, Real](limit(partial(a))), mask_seq_sum(a, s))

        forall(i: Nat) {
            abs_diff_seq(mask_seq_sum(a, s), constant[Nat, Real](limit(partial(a))), i) =
                abs_diff_seq(constant[Nat, Real](limit(partial(a))), mask_seq_sum(a, s), i)
        }
        abs_diff_seq(mask_seq_sum(a, s), constant[Nat, Real](limit(partial(a)))) =
            abs_diff_seq(constant[Nat, Real](limit(partial(a))), mask_seq_sum(a, s))

        vanishing_diff(constant[Nat, Real](limit(partial(a))), mask_seq_sum(a, s))

        if converges(constant[Nat, Real](limit(partial(a)))) and
           vanishing_diff(constant[Nat, Real](limit(partial(a))), mask_seq_sum(a, s)) {
            converges(mask_seq_sum(a, s)) and
                limit(constant[Nat, Real](limit(partial(a)))) = limit(mask_seq_sum(a, s))
        }

        converges(mask_seq_sum(a, s)) and
            limit(constant[Nat, Real](limit(partial(a)))) = limit(mask_seq_sum(a, s))

        limit(constant[Nat, Real](limit(partial(a)))) = limit(partial(a))
        limit(mask_seq_sum(a, s)) = limit(partial(a))

        converges_to(mask_seq_sum(a, s), limit(partial(a)))
    }
}

/// For m > n, the element at index m in the mask for singleton {n} is zero.
/// Proof: Since m != n, the singleton set {n} does not contain m,
/// so by the definition of mask, mask(a, {n}, m) = 0.
theorem mask_singleton_zero_after(a: Nat -> Real, n: Nat, m: Nat) {
    n < m
    implies
    mask(a, Set[Nat].singleton(n), m) = Real.0
} by {
    not n < m or m != n
    m != n
    not (m ∈ Set[Nat].singleton(n))
    filter_seq(a, Set.singleton(n).contains, m) = mask(a, Set.singleton(n), m)
    filter_seq(a, Set.singleton(n).contains, m) = Real.0
}

/// Helper function that is a(n) at index n, and 0 elsewhere.
define point_mass(a: Nat -> Real, n: Nat, i: Nat) -> Real {
    if i = n { a(n) } else { Real.0 }
}

/// The partial sum of a point mass at n equals a(n).
theorem partial_point_mass(a: Nat -> Real, n: Nat) {
    partial(point_mass(a, n), n.suc) = a(n)
} by {
    define p(m: Nat) -> Bool {
        partial(point_mass(a, m), m.suc) = a(m)
    }

    // Base case: m = 0
    point_mass(a, Nat.0, Nat.0) = a(Nat.0)
    partial(point_mass(a, Nat.0), Nat.1) = a(Nat.0)
    p(Nat.0)

    // Inductive step
    forall(m: Nat) {
        if p(m) {
            from list import partial_split_last
            from real.real_series import partial_all_zeros

            // For m.suc, we need partial(point_mass(a, m.suc), m.suc.suc) = a(m.suc)
            // point_mass(a, m.suc, i) = a(m.suc) if i = m.suc, else 0

            // Use partial_split_last
            partial(point_mass(a, m.suc), m.suc.suc) =
                partial(point_mass(a, m.suc), m.suc) + point_mass(a, m.suc, m.suc)

            point_mass(a, m.suc, m.suc) = a(m.suc)

            // For i < m.suc, point_mass(a, m.suc, i) != a(m.suc) since i != m.suc
            forall(i: Nat) {
                if i < m.suc {
                    point_mass(a, m.suc, i) = Real.0
                }
            }

            partial(point_mass(a, m.suc), m.suc) = Real.0
            partial(point_mass(a, m.suc), m.suc.suc) = Real.0 + a(m.suc)
            partial(point_mass(a, m.suc), m.suc.suc) = a(m.suc)
            p(m.suc)
        }
    }

    p(n)
}

/// mask for singleton equals point_mass pointwise.
theorem mask_singleton_eq_point_mass(a: Nat -> Real, n: Nat, i: Nat) {
    mask(a, Set[Nat].singleton(n), i) = point_mass(a, n, i)
} by {
    if i = n {
        Set[Nat].singleton(n).contains(i)
        mask(a, Set[Nat].singleton(n), i) = a(i)
        point_mass(a, n, i) = a(n)
        mask(a, Set[Nat].singleton(n), i) = point_mass(a, n, i)
    } else {
        not (i = n)
        not Set[Nat].singleton(n).contains(i)
        mask(a, Set[Nat].singleton(n), i) = Real.0
        point_mass(a, n, i) = Real.0
        mask(a, Set[Nat].singleton(n), i) = point_mass(a, n, i)
    }
}

/// The partial sum of the mask for singleton {n} at index n+1 equals a(n).
theorem mask_singleton_base_case(a: Nat -> Real, n: Nat) {
    partial(mask(a, Set[Nat].singleton(n)), n.suc) = a(n)
} by {
    from list import partial_pointwise_eq

    forall(i: Nat) {
        if i < n.suc {
            mask(a, Set[Nat].singleton(n), i) = point_mass(a, n, i)
        }
    }

    partial(mask(a, Set[Nat].singleton(n)), n.suc) = partial(point_mass(a, n), n.suc)

    partial(point_mass(a, n), n.suc) = a(n)

    partial(mask(a, Set[Nat].singleton(n)), n.suc) = a(n)
}

/// The partial sum of the mask for singleton {n} eventually equals a(n).
theorem mask_singleton_eventually_constant(a: Nat -> Real, n: Nat, m: Nat) {
    n.suc <= m
    implies
    partial(mask(a, Set[Nat].singleton(n)), m) = a(n)
} by {
    if n.suc <= m {
        define p(d: Nat) -> Bool {
            partial(mask(a, Set[Nat].singleton(n)), n.suc + d) = a(n)
        }

        // Base: d = 0
        partial(mask(a, Set[Nat].singleton(n)), n.suc) = a(n)
        p(Nat.0)

        // Step: p(d) => p(d.suc)
        forall(d: Nat) {
            if p(d) {
                mask(a, Set[Nat].singleton(n), n.suc + d) = Real.0

                partial(mask(a, Set[Nat].singleton(n)), (n.suc + d).suc) =
                    partial(mask(a, Set[Nat].singleton(n)), n.suc + d) +
                    mask(a, Set[Nat].singleton(n), n.suc + d)

                partial(mask(a, Set[Nat].singleton(n)), n.suc + d.suc) =
                    partial(mask(a, Set[Nat].singleton(n)), n.suc + d) + Real.0

                partial(mask(a, Set[Nat].singleton(n)), n.suc + d) + Real.0 = partial(mask(a, Set[Nat].singleton(n)), n.suc + d)
                p(d)
                partial(mask(a, Set[Nat].singleton(n)), n.suc + d.suc) = a(n)
                p(d.suc)
            }
        }

        // Extract d such that m = n.suc + d
        let d: Nat satisfy {
            n.suc + d = m
        }
        p(d)
        partial(mask(a, Set[Nat].singleton(n)), m) = a(n)
    }
}

/// For a singleton set {n}, the limit of the partial sum of the mask equals a(n).
theorem mask_singleton_limit(a: Nat -> Real, n: Nat) {
    limit(partial(mask(a, Set[Nat].singleton(n)))) = a(n)
} by {

        forall(eps: Real) {
            if eps.is_positive {
                forall(m: Nat) {
                    if n.suc <= m {
                    partial(mask(a, Set[Nat].singleton(n)), m) = a(n)
                        partial(mask(a, Set[Nat].singleton(n)), m).is_close(a(n), eps)
                    }
                }
                forall(i: Nat) {
                    n.suc <= i implies partial(mask(a, Set[Nat].singleton(n)), i).is_close(a(n), eps)
                }
                exists(k0: Nat) {
                    n.suc <= k0 and not partial(mask(a, Set[Nat].singleton(n)), k0).is_close(a(n), eps)
                } or tail_bound(partial(mask(a, Set[Nat].singleton(n))), a(n), n.suc, eps)
                tail_bound(partial(mask(a, Set[Nat].singleton(n))), a(n), n.suc, eps)
                let tail_n: Nat satisfy {
                    tail_bound(partial(mask(a, Set[Nat].singleton(n))), a(n), tail_n, eps)
                }
                exists(tail_n0: Nat) {
                    tail_bound(partial(mask(a, Set[Nat].singleton(n))), a(n), tail_n0, eps)
                }
            }
        }

    forall(eps: Real) {
        eps.is_positive implies exists(tail_n0: Nat) {
            tail_bound(partial(mask(a, Set[Nat].singleton(n))), a(n), tail_n0, eps)
        }
    }

    exists(k0: Real) {
        k0.is_positive and forall(x0: Nat) {
            not tail_bound(partial(mask(a, Set[Nat].singleton(n))), a(n), x0, k0)
        }
    } or converges_to(partial(mask(a, Set[Nat].singleton(n))), a(n))
    converges_to(partial(mask(a, Set[Nat].singleton(n))), a(n))
    converges(partial(mask(a, Set[Nat].singleton(n))))
    limit(partial(mask(a, Set[Nat].singleton(n)))) = a(n)
}

/// For an absolutely convergent series a, any function p and natural number k,
/// the partial sum of compose(a, p) up to k equals the limit of the partial sum
/// of the mask for image_of_range(p, k).
/// For any n, the partial sum of n zeros is zero.
theorem partial_constant_zero(n: Nat) {
    partial(constant[Nat, Real](Real.0), n) = Real.0
} by {
    define q(m: Nat) -> Bool {
        partial(constant[Nat, Real](Real.0), m) = Real.0
    }


    // Base case
    q(Nat.0)

    // Inductive step
    forall(m: Nat) {
        if q(m) {
            from list import partial_split_last
            partial(constant[Nat, Real](Real.0), m.suc) =
                partial(constant[Nat, Real](Real.0), m) + constant[Nat, Real](Real.0, m)
            constant[Nat, Real](Real.0, m) = Real.0
            partial(constant[Nat, Real](Real.0), m.suc) = Real.0 + Real.0
            partial(constant[Nat, Real](Real.0), m.suc) = Real.0
            q(m.suc)
        }
    }

    q(n)
}

/// The partial sum sequence of the constant zero sequence is itself constant zero.
theorem partial_constant_zero_is_constant {
    partial(constant[Nat, Real](Real.0)) = constant[Nat, Real](Real.0)
} by {
    forall(n: Nat) {
        partial(constant[Nat, Real](Real.0), n) = Real.0
        partial(constant[Nat, Real](Real.0), n) = constant[Nat, Real](Real.0, n)
    }
}

theorem partial_compose_eq_mask_sum(a: Nat -> Real, p: Nat -> Nat, k: Nat) {
    absolutely_converges(a) and is_injective(p)
    implies
    partial(compose(a, p), k) = limit(partial(mask(a, image_of_range(p, k))))
} by {
    from nat import alt_induction, not_lt_zero

    if absolutely_converges(a) and is_injective(p) {
        define q(m: Nat) -> Bool {
            absolutely_converges(a) and is_injective(p)
            implies
            partial(compose(a, p), m) = limit(partial(mask(a, image_of_range(p, m))))
        }

        // Base case: m = 0
        // For m = 0, image_of_range(p, 0) is empty, so mask is all zeros
        forall(n: Nat) {
            if image_of_range(p, Nat.0).contains(n) {
                image_of_range_eq_set_image(p, Nat.0)
                set_image_contains_witness(nat_lt_set(Nat.0), p, n)
                let i: Nat satisfy {
                    nat_lt_set(Nat.0).contains(i) and n = p(i)
                }
                nat_lt_set_contains_eq(Nat.0, i)
                i < Nat.0
                false
            }
            not image_of_range(p, Nat.0).contains(n)
            mask(a, image_of_range(p, Nat.0), n) = Real.0
        }
        forall(n: Nat) {
            mask(a, image_of_range(p, Nat.0), n) = Real.0
            constant[Nat, Real](Real.0, n) = Real.0
            mask(a, image_of_range(p, Nat.0), n) = constant[Nat, Real](Real.0, n)
        }
        mask(a, image_of_range(p, Nat.0)) = constant[Nat, Real](Real.0)

        partial(compose(a, p), Nat.0) = Real.0
        partial(mask(a, image_of_range(p, Nat.0))) = partial(constant[Nat, Real](Real.0))

        // Use helper theorem to show partial(constant zero) = constant zero
        partial(constant[Nat, Real](Real.0)) = constant[Nat, Real](Real.0)
        partial(mask(a, image_of_range(p, Nat.0))) = constant[Nat, Real](Real.0)

        // Now apply const_limit
        limit(constant[Nat, Real](Real.0)) = Real.0
        limit(partial(mask(a, image_of_range(p, Nat.0)))) = Real.0
        partial(compose(a, p), Nat.0) = limit(partial(mask(a, image_of_range(p, Nat.0))))
        q(Nat.0)

        // Inductive step: assume q(m), prove q(m.suc)
        forall(m: Nat) {
            if q(m) and absolutely_converges(a) and is_injective(p) {
                from list import partial_split_last

                // partial(compose(a, p), m.suc) = partial(compose(a, p), m) + compose(a, p, m)
                partial(compose(a, p), m.suc) = partial(compose(a, p), m) + compose(a, p, m)
                compose(a, p, m) = a(p(m))

                // By IH: partial(compose(a, p), m) = limit(partial(mask(a, image_of_range(p, m))))
                q(m)
                partial(compose(a, p), m) = limit(partial(mask(a, image_of_range(p, m))))

                // Use helper lemmas
                image_of_range_disjoint_from_next(p, m)
                image_of_range(p, m).is_disjoint(Set[Nat].singleton(p(m)))

                image_of_range_suc_eq_union_singleton(p, m)
                image_of_range(p, m.suc) = image_of_range(p, m).union(Set[Nat].singleton(p(m)))

                // Use mask_disjoint_union_sum
                limit(partial(mask(a, image_of_range(p, m.suc)))) =
                    limit(partial(mask(a, image_of_range(p, m)))) +
                    limit(partial(mask(a, Set[Nat].singleton(p(m)))))

                // Use mask_singleton_limit
                limit(partial(mask(a, Set[Nat].singleton(p(m))))) = a(p(m))

                // Combine
                limit(partial(mask(a, image_of_range(p, m.suc)))) =
                    limit(partial(mask(a, image_of_range(p, m)))) + a(p(m))

                partial(compose(a, p), m.suc) =
                    limit(partial(mask(a, image_of_range(p, m)))) + a(p(m))

                partial(compose(a, p), m.suc) = limit(partial(mask(a, image_of_range(p, m.suc))))
                q(m.suc)
            }
        }

        forall(n: Nat) {
            q(n) implies q(n.suc)
        }
        forall(n: Nat) { q(n) }
        q(k)
        partial(compose(a, p), k) = limit(partial(mask(a, image_of_range(p, k))))
    }
}

/// For an absolutely convergent series a and a bijection p: Nat -> Nat,
/// the rearranged series compose(a, p) is also absolutely convergent.
theorem riemann_rearrangement_absolutely_converges(a: Nat -> Real, p: Nat -> Nat) {
    absolutely_converges(a) and is_bijection(p)
    implies
    absolutely_converges(compose(a, p))
} by {
    if absolutely_converges(a) and is_bijection(p) {
        is_bijection(p)
        is_injective(p)
        is_surjective(p)

        is_increasing(image_of_range(p))

        seq_union(image_of_range(p)) = Set[Nat].universal_set

        // abs_fn(compose(a, p)) = compose(abs_fn(a), p)
        forall(n: Nat) {
            compose(abs_fn(a), p)(n) = abs_fn(a)(p(n))
            abs_fn(a)(p(n)) = a(p(n)).abs
            compose(a, p)(n) = a(p(n))
            abs_fn(compose(a, p))(n) = compose(a, p)(n).abs
            compose(a, p)(n).abs = a(p(n)).abs
            abs_fn(compose(a, p))(n) = compose(abs_fn(a), p)(n)
        }
        abs_fn(compose(a, p)) = compose(abs_fn(a), p)

        // abs_fn(a) is also absolutely convergent (since abs(abs(x)) = abs(x))
        forall(n: Nat) {
            abs_fn(abs_fn(a))(n) = abs_fn(a)(n).abs
            abs_fn(a)(n).abs = abs_fn(a)(n)
            abs_fn(abs_fn(a))(n) = abs_fn(a)(n)
        }
        abs_fn(abs_fn(a)) = abs_fn(a)
        partial(abs_fn(abs_fn(a))) = partial(abs_fn(a))

        absolutely_converges(a)
        converges(partial(abs_fn(a)))
        converges(partial(abs_fn(abs_fn(a))))
        absolutely_converges(abs_fn(a))

        // Show partial(compose(abs_fn(a), p)) = mask_seq_sum(abs_fn(a), image_of_range(p))
        forall(k: Nat) {
            partial(compose(abs_fn(a), p), k) = limit(partial(mask(abs_fn(a), image_of_range(p, k))))
            mask_seq_sum(abs_fn(a), image_of_range(p), k) = limit(partial(mask(abs_fn(a), image_of_range(p, k))))
            partial(compose(abs_fn(a), p), k) = mask_seq_sum(abs_fn(a), image_of_range(p), k)
        }
        partial(compose(abs_fn(a), p)) = mask_seq_sum(abs_fn(a), image_of_range(p))

        // Since abs_fn(compose(a, p)) = compose(abs_fn(a), p)
        partial(abs_fn(compose(a, p))) = partial(compose(abs_fn(a), p))
        partial(abs_fn(compose(a, p))) = mask_seq_sum(abs_fn(a), image_of_range(p))

        // By increasing_universal_union_mask_limit on abs_fn(a)
        converges_to(mask_seq_sum(abs_fn(a), image_of_range(p)), limit(partial(abs_fn(a))))

        converges_to(partial(abs_fn(compose(a, p))), limit(partial(abs_fn(a))))
        converges(partial(abs_fn(compose(a, p))))
        absolutely_converges(compose(a, p))
    }
}

/// Riemann Rearrangement Theorem:
/// For an absolutely convergent series a and a bijection p: Nat -> Nat,
/// the rearranged series compose(a, p) converges to the same limit.
theorem riemann_rearrangement_theorem(a: Nat -> Real, p: Nat -> Nat) {
    absolutely_converges(a) and is_bijection(p)
    implies
    converges_to(partial(compose(a, p)), limit(partial(a)))
} by {
    if absolutely_converges(a) and is_bijection(p) {
        is_bijection(p)
        is_injective(p)
        is_surjective(p)

        is_increasing(image_of_range(p))

        seq_union(image_of_range(p)) = Set[Nat].universal_set

        // Show partial(compose(a, p)) = mask_seq_sum(a, image_of_range(p))
        forall(k: Nat) {
            partial(compose(a, p), k) = limit(partial(mask(a, image_of_range(p, k))))
            mask_seq_sum(a, image_of_range(p), k) = limit(partial(mask(a, image_of_range(p, k))))
            partial(compose(a, p), k) = mask_seq_sum(a, image_of_range(p), k)
        }
        partial(compose(a, p)) = mask_seq_sum(a, image_of_range(p))

        // Use increasing_universal_union_mask_limit
        converges_to(mask_seq_sum(a, image_of_range(p)), limit(partial(a)))

        converges_to(partial(compose(a, p)), limit(partial(a)))
    }
}
