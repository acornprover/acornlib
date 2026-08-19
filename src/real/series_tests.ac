/// Convergence tests for real series: the alternating series test (Leibniz),
/// the ratio test, and statements of the root and integral tests.

from rat import Rat
from nat import Nat, add_sub, divides_zero, two_divides_suc_iff, one_pow, pow_add, division_theorem, alt_induction, lt_suc_right, not_lt_zero
from list import partial, sum, map, List
from algebra.ring.ring import alternating_sign, alternating_sign_zero, alternating_sign_suc, alternating_sign_eq_neg_one_pow, alternating_sign_parity
from real.real_ring import Real, converges, limit, converges_to, mul_abs
from real.real_base import lte_trans, lte_add_right, lte_add_left, lte_lt_trans, lt_trans, neg_pos_is_neg, neg_neg, close_imp_bounds, rat_between_reals, not_lt_imp_gte, lt_add_right, lte_abs
from real.real_seq import tail_bound, tail_bound_implies_is_close, converges_imp_converges_to
from real.real_series import is_lower_bound, is_decreasing_seq, decreasing_convergent_bounded_by_limit, partial_suc, partial_zero, diff_partial, partial_tail_sub, tail, mul_seq, partial_mul_seq_comm, abs_pow
from real.abs_conv import absolutely_converges, abs_fn, absolutely_converges_imp_converges
from real.exp import always_nonzero, has_ratio_bound, has_ratio_bound_eventually, eventual_ratio_bound_implies_abs_conv
from real.series_cauchy import is_cauchy_series, cauchy_series_imp_converges
from real import two_mul_suc
from real.limits import vanishes

numerals Real

/// The alternating term (−1)ⁿ · a(n) of a series.
define alt_term(a: Nat -> Real, n: Nat) -> Real {
    alternating_sign[Real](n) * a(n)
}

/// A nonnegative real is its own absolute value.
theorem nonneg_imp_abs_eq(x: Real) {
    Real.0 <= x implies x.abs = x
} by {
    if Real.0 <= x {
        not x.is_negative
        x.abs = x
    }
}

/// The absolute value of minus one is one.
theorem abs_neg_one {
    (-Real.1).abs = Real.1
} by {
    Real.1.is_positive
    neg_pos_is_neg(Real.1)
    (-Real.1).is_negative
    (-Real.1).abs = -(-Real.1)
    neg_neg(Real.1)
    -(-Real.1) = Real.1
    (-Real.1).abs = Real.1
}

/// The alternating sign has absolute value one.
theorem alt_sign_abs_one(n: Nat) {
    alternating_sign[Real](n).abs = Real.1
} by {
    alternating_sign_eq_neg_one_pow[Real](n)
    alternating_sign[Real](n) = (-Real.1).pow(n)
    abs_pow(-Real.1, n)
    (-Real.1).abs.pow(n) = (-Real.1).pow(n).abs
    abs_neg_one
    (-Real.1).abs = Real.1
    one_pow[Real](n)
    Real.1.pow(n) = Real.1
    alternating_sign[Real](n).abs = Real.1
}

/// The alternating sign at a sum of indices is the product of the signs.
theorem alt_sign_add(n: Nat, m: Nat) {
    alternating_sign[Real](n + m) = alternating_sign[Real](n) * alternating_sign[Real](m)
} by {
    alternating_sign_eq_neg_one_pow[Real](n + m)
    alternating_sign[Real](n + m) = (-Real.1).pow(n + m)
    pow_add[Real](-Real.1, n, m)
    (-Real.1).pow(n + m) = (-Real.1).pow(n) * (-Real.1).pow(m)
    alternating_sign_eq_neg_one_pow[Real](n)
    alternating_sign[Real](n) = (-Real.1).pow(n)
    alternating_sign_eq_neg_one_pow[Real](m)
    alternating_sign[Real](m) = (-Real.1).pow(m)
    alternating_sign[Real](n + m) = alternating_sign[Real](n) * alternating_sign[Real](m)
}

/// The tail of a decreasing sequence is decreasing.
theorem tail_decreasing_seq(a: Nat -> Real, k: Nat) {
    is_decreasing_seq(a) implies is_decreasing_seq(tail(a, k))
} by {
    if is_decreasing_seq(a) {
        forall(n: Nat) {
            tail(a, k)(n.suc) = a(k + n.suc)
            tail(a, k)(n) = a(k + n)
            k + n.suc = (k + n).suc
            a((k + n).suc) <= a(k + n)
            tail(a, k)(n.suc) <= tail(a, k)(n)
        }
        is_decreasing_seq(tail(a, k))
    }
}

/// A lower bound of a sequence is a lower bound of every tail.
theorem tail_lower_bound_seq(a: Nat -> Real, k: Nat, lb: Real) {
    is_lower_bound(a, lb) implies is_lower_bound(tail(a, k), lb)
} by {
    if is_lower_bound(a, lb) {
        forall(n: Nat) {
            tail(a, k)(n) = a(k + n)
            lb <= a(k + n)
            lb <= tail(a, k)(n)
        }
        is_lower_bound(tail(a, k), lb)
    }
}

/// The zeroth element of a tail is the element at the shift.
theorem tail_zero_at(a: Nat -> Real, k: Nat) {
    tail(a, k)(Nat.0) = a(k)
} by {
    tail(a, k)(Nat.0) = a(k + Nat.0)
    k + Nat.0 = k
    tail(a, k)(Nat.0) = a(k)
}

/// The tail of the alternating sequence is the alternating sequence of the
/// tail, up to the sign at the shift.
theorem alt_term_tail(a: Nat -> Real, k: Nat) {
    tail(alt_term(a), k) = mul_seq(alternating_sign[Real](k), alt_term(tail(a, k)))
} by {
    forall(j: Nat) {
        tail(alt_term(a), k, j) = alt_term(a)(k + j)
        alt_term(a)(k + j) = alternating_sign[Real](k + j) * a(k + j)
        alt_sign_add(k, j)
        alternating_sign[Real](k + j) = alternating_sign[Real](k) * alternating_sign[Real](j)
        tail(a, k, j) = a(k + j)
        alt_term(tail(a, k), j) = alternating_sign[Real](j) * tail(a, k, j)
        alternating_sign[Real](k + j) * a(k + j) = alternating_sign[Real](k) * (alternating_sign[Real](j) * a(k + j))
        alternating_sign[Real](k) * alt_term(tail(a, k), j) = alternating_sign[Real](k) * (alternating_sign[Real](j) * a(k + j))
        mul_seq(alternating_sign[Real](k), alt_term(tail(a, k)), j) = alternating_sign[Real](k) * alt_term(tail(a, k), j)
        tail(alt_term(a), k, j) = mul_seq(alternating_sign[Real](k), alt_term(tail(a, k)), j)
    }
}

/// The even-index bound predicate: W(2k) <= c(0) − c(2k).
define alt_even_ub_pred(c: Nat -> Real, k: Nat) -> Bool {
    partial(alt_term(c), Nat.2 * k) <= c(Nat.0) - c(Nat.2 * k)
}

/// The even-index nonnegativity predicate: 0 <= W(2k).
define alt_even_lb_pred(c: Nat -> Real, k: Nat) -> Bool {
    Real.0 <= partial(alt_term(c), Nat.2 * k)
}

/// The even-index partial sums of an alternating decreasing nonnegative
/// sequence step from W(2k) <= c(0) − c(2k) to W(2(k+1)) <= c(0) − c(2(k+1)).
theorem alt_even_ub_suc(c: Nat -> Real, k: Nat) {
    is_decreasing_seq(c) and is_lower_bound(c, Real.0)
    and alt_even_ub_pred(c, k)
    implies alt_even_ub_pred(c, k.suc)
} by {
    if is_decreasing_seq(c) and is_lower_bound(c, Real.0)
        and alt_even_ub_pred(c, k) {
        partial(alt_term(c), Nat.2 * k) <= c(Nat.0) - c(Nat.2 * k)
        // W(2k+2) = W(2k) + c(2k) − c(2k+1).
        partial_suc(alt_term(c), Nat.2 * k)
        partial(alt_term(c), (Nat.2 * k).suc) = partial(alt_term(c), Nat.2 * k) + alt_term(c)(Nat.2 * k)
        alternating_sign_parity[Real](Nat.2 * k)
        alternating_sign[Real](Nat.2 * k) = Real.1
        alt_term(c)(Nat.2 * k) = alternating_sign[Real](Nat.2 * k) * c(Nat.2 * k)
        alt_term(c)(Nat.2 * k) = c(Nat.2 * k)
        partial(alt_term(c), (Nat.2 * k).suc) = partial(alt_term(c), Nat.2 * k) + c(Nat.2 * k)
        partial_suc(alt_term(c), (Nat.2 * k).suc)
        partial(alt_term(c), (Nat.2 * k).suc.suc) = partial(alt_term(c), (Nat.2 * k).suc) + alt_term(c)((Nat.2 * k).suc)
        alternating_sign_parity[Real]((Nat.2 * k).suc)
        alternating_sign[Real]((Nat.2 * k).suc) = -Real.1
        alt_term(c)((Nat.2 * k).suc) = -c((Nat.2 * k).suc)
        partial(alt_term(c), (Nat.2 * k).suc.suc) = partial(alt_term(c), (Nat.2 * k).suc) - c((Nat.2 * k).suc)
        partial(alt_term(c), (Nat.2 * k).suc.suc) = partial(alt_term(c), Nat.2 * k) + c(Nat.2 * k) - c((Nat.2 * k).suc)
        two_mul_suc(k)
        Nat.2 * k.suc = (Nat.2 * k).suc.suc
        partial(alt_term(c), Nat.2 * k.suc) = partial(alt_term(c), Nat.2 * k) + c(Nat.2 * k) - c((Nat.2 * k).suc)
        c((Nat.2 * k).suc) <= c(Nat.2 * k)
        c(Nat.0) - c(Nat.2 * k) + c(Nat.2 * k) = c(Nat.0)
        lte_add_right(partial(alt_term(c), Nat.2 * k), c(Nat.0) - c(Nat.2 * k), c(Nat.2 * k))
        partial(alt_term(c), Nat.2 * k) + c(Nat.2 * k) <= c(Nat.0) - c(Nat.2 * k) + c(Nat.2 * k)
        partial(alt_term(c), Nat.2 * k) + c(Nat.2 * k) <= c(Nat.0)
        lte_add_right(partial(alt_term(c), Nat.2 * k) + c(Nat.2 * k), c(Nat.0), -c((Nat.2 * k).suc))
        partial(alt_term(c), Nat.2 * k) + c(Nat.2 * k) + -c((Nat.2 * k).suc) <= c(Nat.0) + -c((Nat.2 * k).suc)
        partial(alt_term(c), Nat.2 * k) + c(Nat.2 * k) - c((Nat.2 * k).suc) <= c(Nat.0) - c((Nat.2 * k).suc)
        partial(alt_term(c), Nat.2 * k.suc) <= c(Nat.0) - c((Nat.2 * k).suc)
        c((Nat.2 * k).suc.suc) <= c((Nat.2 * k).suc)
        -c((Nat.2 * k).suc) <= -c((Nat.2 * k).suc.suc)
        lte_add_left(-c((Nat.2 * k).suc), -c((Nat.2 * k).suc.suc), c(Nat.0))
        c(Nat.0) + -c((Nat.2 * k).suc) <= c(Nat.0) + -c((Nat.2 * k).suc.suc)
        c(Nat.0) - c((Nat.2 * k).suc) <= c(Nat.0) - c((Nat.2 * k).suc.suc)
        lte_trans(partial(alt_term(c), Nat.2 * k.suc), c(Nat.0) - c((Nat.2 * k).suc), c(Nat.0) - c((Nat.2 * k).suc.suc))
        partial(alt_term(c), Nat.2 * k.suc) <= c(Nat.0) - c((Nat.2 * k).suc.suc)
        c((Nat.2 * k).suc.suc) = c(Nat.2 * k.suc)
        partial(alt_term(c), Nat.2 * k.suc) <= c(Nat.0) - c(Nat.2 * k.suc)
        alt_even_ub_pred(c, k.suc)
    }
}

/// The even-index bound steps at every index: from W(2k) <= c(0) − c(2k) to
/// W(2(k+1)) <= c(0) − c(2(k+1)).
theorem alt_even_ub_step(c: Nat -> Real) {
    is_decreasing_seq(c) and is_lower_bound(c, Real.0)
    implies forall(k: Nat) {
        alt_even_ub_pred(c, k) implies alt_even_ub_pred(c, k.suc)
    }
} by {
    if is_decreasing_seq(c) and is_lower_bound(c, Real.0) {
        forall(k: Nat) {
            alt_even_ub_pred(c, k) implies alt_even_ub_pred(c, k.suc)
        }
    }
}

/// The partial sums of an alternating decreasing nonnegative sequence stay
/// below c(0) − c(2q) at even indices.
theorem alt_even_ub(c: Nat -> Real, q: Nat) {
    is_decreasing_seq(c) and is_lower_bound(c, Real.0)
    implies partial(alt_term(c), Nat.2 * q) <= c(Nat.0) - c(Nat.2 * q)
} by {
    if is_decreasing_seq(c) and is_lower_bound(c, Real.0) {
        // Base case: q = 0.
        partial(alt_term(c), Nat.0) = Real.0
        Real.0 <= c(Nat.0) - c(Nat.0)
        partial(alt_term(c), Nat.0) <= c(Nat.0) - c(Nat.0)
        alt_even_ub_pred(c, Nat.0)
        // Inductive step.
        alt_even_ub_step(c)
        forall(k: Nat) {
            alt_even_ub_pred(c, k) implies alt_even_ub_pred(c, k.suc)
        }
        alt_induction(alt_even_ub_pred(c))
        forall(k: Nat) { alt_even_ub_pred(c, k) }
        alt_even_ub_pred(c, q)
        partial(alt_term(c), Nat.2 * q) <= c(Nat.0) - c(Nat.2 * q)
    }
}

/// The even-index partial sums of an alternating decreasing nonnegative
/// sequence step from 0 <= W(2k) to 0 <= W(2(k+1)).
theorem alt_even_lb_suc(c: Nat -> Real, k: Nat) {
    is_decreasing_seq(c) and is_lower_bound(c, Real.0)
    and alt_even_lb_pred(c, k)
    implies alt_even_lb_pred(c, k.suc)
} by {
    if is_decreasing_seq(c) and is_lower_bound(c, Real.0)
        and alt_even_lb_pred(c, k) {
        Real.0 <= partial(alt_term(c), Nat.2 * k)
        // W(2k+2) = W(2k) + c(2k) − c(2k+1) >= 0.
        partial_suc(alt_term(c), Nat.2 * k)
        partial(alt_term(c), (Nat.2 * k).suc) = partial(alt_term(c), Nat.2 * k) + alt_term(c)(Nat.2 * k)
        alternating_sign_parity[Real](Nat.2 * k)
        alternating_sign[Real](Nat.2 * k) = Real.1
        alt_term(c)(Nat.2 * k) = alternating_sign[Real](Nat.2 * k) * c(Nat.2 * k)
        alt_term(c)(Nat.2 * k) = c(Nat.2 * k)
        partial(alt_term(c), (Nat.2 * k).suc) = partial(alt_term(c), Nat.2 * k) + c(Nat.2 * k)
        partial_suc(alt_term(c), (Nat.2 * k).suc)
        partial(alt_term(c), (Nat.2 * k).suc.suc) = partial(alt_term(c), (Nat.2 * k).suc) + alt_term(c)((Nat.2 * k).suc)
        alternating_sign_parity[Real]((Nat.2 * k).suc)
        alternating_sign[Real]((Nat.2 * k).suc) = -Real.1
        alt_term(c)((Nat.2 * k).suc) = -c((Nat.2 * k).suc)
        partial(alt_term(c), (Nat.2 * k).suc.suc) = partial(alt_term(c), (Nat.2 * k).suc) - c((Nat.2 * k).suc)
        two_mul_suc(k)
        Nat.2 * k.suc = (Nat.2 * k).suc.suc
        partial(alt_term(c), Nat.2 * k.suc) = partial(alt_term(c), Nat.2 * k) + c(Nat.2 * k) - c((Nat.2 * k).suc)
        c((Nat.2 * k).suc) <= c(Nat.2 * k)
        lte_add_right(c((Nat.2 * k).suc), c(Nat.2 * k), -c((Nat.2 * k).suc))
        c((Nat.2 * k).suc) + -c((Nat.2 * k).suc) <= c(Nat.2 * k) + -c((Nat.2 * k).suc)
        Real.0 <= c(Nat.2 * k) - c((Nat.2 * k).suc)
        lte_add_right(Real.0, partial(alt_term(c), Nat.2 * k), c(Nat.2 * k) - c((Nat.2 * k).suc))
        Real.0 + (c(Nat.2 * k) - c((Nat.2 * k).suc)) <= partial(alt_term(c), Nat.2 * k) + (c(Nat.2 * k) - c((Nat.2 * k).suc))
        Real.0 <= partial(alt_term(c), Nat.2 * k) + (c(Nat.2 * k) - c((Nat.2 * k).suc))
        partial(alt_term(c), Nat.2 * k) + c(Nat.2 * k) - c((Nat.2 * k).suc) = partial(alt_term(c), Nat.2 * k) + (c(Nat.2 * k) - c((Nat.2 * k).suc))
        Real.0 <= partial(alt_term(c), Nat.2 * k.suc)
        alt_even_lb_pred(c, k.suc)
    }
}

/// The even-index nonnegativity steps at every index: from 0 <= W(2k) to
/// 0 <= W(2(k+1)).
theorem alt_even_lb_step(c: Nat -> Real) {
    is_decreasing_seq(c) and is_lower_bound(c, Real.0)
    implies forall(k: Nat) {
        alt_even_lb_pred(c, k) implies alt_even_lb_pred(c, k.suc)
    }
} by {
    if is_decreasing_seq(c) and is_lower_bound(c, Real.0) {
        forall(k: Nat) {
            alt_even_lb_pred(c, k) implies alt_even_lb_pred(c, k.suc)
        }
    }
}

/// The partial sums of an alternating decreasing nonnegative sequence are
/// nonnegative at even indices.
theorem alt_even_lb(c: Nat -> Real, q: Nat) {
    is_decreasing_seq(c) and is_lower_bound(c, Real.0)
    implies Real.0 <= partial(alt_term(c), Nat.2 * q)
} by {
    if is_decreasing_seq(c) and is_lower_bound(c, Real.0) {
        // Base case: q = 0.
        partial(alt_term(c), Nat.0) = Real.0
        Real.0 <= partial(alt_term(c), Nat.0)
        alt_even_lb_pred(c, Nat.0)
        // Inductive step.
        alt_even_lb_step(c)
        forall(k: Nat) {
            alt_even_lb_pred(c, k) implies alt_even_lb_pred(c, k.suc)
        }
        alt_induction(alt_even_lb_pred(c))
        forall(k: Nat) { alt_even_lb_pred(c, k) }
        alt_even_lb_pred(c, q)
        Real.0 <= partial(alt_term(c), Nat.2 * q)
    }
}

/// The partial sums of an alternating decreasing nonnegative sequence stay
/// below c(0) at odd indices.
theorem alt_odd_ub(c: Nat -> Real, q: Nat) {
    is_decreasing_seq(c) and is_lower_bound(c, Real.0)
    implies partial(alt_term(c), Nat.2 * q + Nat.1) <= c(Nat.0)
} by {
    if is_decreasing_seq(c) and is_lower_bound(c, Real.0) {
        alt_even_ub(c, q)
        partial(alt_term(c), Nat.2 * q) <= c(Nat.0) - c(Nat.2 * q)
        partial_suc(alt_term(c), Nat.2 * q)
        partial(alt_term(c), (Nat.2 * q).suc) = partial(alt_term(c), Nat.2 * q) + alt_term(c)(Nat.2 * q)
        alternating_sign_parity[Real](Nat.2 * q)
        alternating_sign[Real](Nat.2 * q) = Real.1
        alt_term(c)(Nat.2 * q) = alternating_sign[Real](Nat.2 * q) * c(Nat.2 * q)
        alt_term(c)(Nat.2 * q) = c(Nat.2 * q)
        partial(alt_term(c), (Nat.2 * q).suc) = partial(alt_term(c), Nat.2 * q) + c(Nat.2 * q)
        lte_add_right(partial(alt_term(c), Nat.2 * q), c(Nat.0) - c(Nat.2 * q), c(Nat.2 * q))
        partial(alt_term(c), Nat.2 * q) + c(Nat.2 * q) <= c(Nat.0) - c(Nat.2 * q) + c(Nat.2 * q)
        c(Nat.0) - c(Nat.2 * q) + c(Nat.2 * q) = c(Nat.0)
        partial(alt_term(c), Nat.2 * q) + c(Nat.2 * q) <= c(Nat.0)
        (Nat.2 * q).suc = Nat.2 * q + Nat.1
        partial(alt_term(c), Nat.2 * q + Nat.1) <= c(Nat.0)
    }
}

/// The partial sums of an alternating decreasing nonnegative sequence stay
/// above c(2q+1) at odd indices.
theorem alt_odd_lb(c: Nat -> Real, q: Nat) {
    is_decreasing_seq(c) and is_lower_bound(c, Real.0)
    implies c(Nat.2 * q + Nat.1) <= partial(alt_term(c), Nat.2 * q + Nat.1)
} by {
    if is_decreasing_seq(c) and is_lower_bound(c, Real.0) {
        alt_even_lb(c, q)
        Real.0 <= partial(alt_term(c), Nat.2 * q)
        partial_suc(alt_term(c), Nat.2 * q)
        partial(alt_term(c), (Nat.2 * q).suc) = partial(alt_term(c), Nat.2 * q) + alt_term(c)(Nat.2 * q)
        alternating_sign_parity[Real](Nat.2 * q)
        alternating_sign[Real](Nat.2 * q) = Real.1
        alt_term(c)(Nat.2 * q) = alternating_sign[Real](Nat.2 * q) * c(Nat.2 * q)
        alt_term(c)(Nat.2 * q) = c(Nat.2 * q)
        partial(alt_term(c), (Nat.2 * q).suc) = partial(alt_term(c), Nat.2 * q) + c(Nat.2 * q)
        Real.0 <= c(Nat.2 * q)
        lte_add_right(Real.0, partial(alt_term(c), Nat.2 * q), c(Nat.2 * q))
        Real.0 + c(Nat.2 * q) <= partial(alt_term(c), Nat.2 * q) + c(Nat.2 * q)
        c(Nat.2 * q) <= partial(alt_term(c), Nat.2 * q) + c(Nat.2 * q)
        c((Nat.2 * q).suc) <= c(Nat.2 * q)
        lte_trans(c((Nat.2 * q).suc), c(Nat.2 * q), partial(alt_term(c), Nat.2 * q) + c(Nat.2 * q))
        c((Nat.2 * q).suc) <= partial(alt_term(c), Nat.2 * q) + c(Nat.2 * q)
        (Nat.2 * q).suc = Nat.2 * q + Nat.1
        partial(alt_term(c), Nat.2 * q + Nat.1) = partial(alt_term(c), Nat.2 * q) + c(Nat.2 * q)
        c(Nat.2 * q + Nat.1) <= partial(alt_term(c), Nat.2 * q + Nat.1)
    }
}

/// Partial sums of an alternating decreasing nonnegative sequence are bounded
/// in absolute value by the first term.
theorem alt_partial_abs_bound(c: Nat -> Real, p: Nat) {
    is_decreasing_seq(c) and is_lower_bound(c, Real.0)
    implies partial(alt_term(c), p).abs <= c(Nat.0)
} by {
    if is_decreasing_seq(c) and is_lower_bound(c, Real.0) {
        if Nat.2.divides(p) {
            let q: Nat satisfy {
                p = Nat.2 * q
            }
            alt_even_ub(c, q)
            partial(alt_term(c), Nat.2 * q) <= c(Nat.0) - c(Nat.2 * q)
            partial(alt_term(c), p) <= c(Nat.0) - c(p)
            Real.0 <= c(p)
            lte_add_right(Real.0, c(p), c(Nat.0))
            Real.0 + c(Nat.0) <= c(p) + c(Nat.0)
            c(Nat.0) <= c(Nat.0) + c(p)
            c(p) + -c(p) = Real.0
            lte_add_right(c(Nat.0), c(Nat.0) + c(p), -c(p))
            c(Nat.0) + -c(p) <= c(Nat.0) + c(p) + -c(p)
            c(Nat.0) + c(p) + -c(p) = c(Nat.0)
            c(Nat.0) - c(p) <= c(Nat.0)
            partial(alt_term(c), p) <= c(Nat.0)
            alt_even_lb(c, q)
            Real.0 <= partial(alt_term(c), Nat.2 * q)
            Real.0 <= partial(alt_term(c), p)
            partial(alt_term(c), p).abs = partial(alt_term(c), p)
            partial(alt_term(c), p).abs <= c(Nat.0)
        } else {
            Nat.0 < Nat.2
            division_theorem(p, Nat.2)
            exists(k0: Nat, k1: Nat) {
                k1 < Nat.2 and p = k0 * Nat.2 + k1
            }
            let (q: Nat, r: Nat) satisfy {
                r < Nat.2 and p = q * Nat.2 + r
            }
            if r = Nat.0 {
                p = q * Nat.2
                Nat.2.divides(p)
                false
            }
            lt_suc_right(r, Nat.1)
            r = Nat.1 or r < Nat.1
            if r < Nat.1 {
                lt_suc_right(r, Nat.0)
                r = Nat.0 or r < Nat.0
                not_lt_zero(r)
                not r < Nat.0
                r = Nat.0
                false
            }
            r = Nat.1
            p = q * Nat.2 + Nat.1
            Nat.2 * q + Nat.1 = p
            alt_odd_ub(c, q)
            partial(alt_term(c), Nat.2 * q + Nat.1) <= c(Nat.0)
            partial(alt_term(c), p) <= c(Nat.0)
            alt_odd_lb(c, q)
            c(Nat.2 * q + Nat.1) <= partial(alt_term(c), Nat.2 * q + Nat.1)
            c(p) <= partial(alt_term(c), p)
            Real.0 <= c(p)
            lte_trans(Real.0, c(p), partial(alt_term(c), p))
            Real.0 <= partial(alt_term(c), p)
            partial(alt_term(c), p).abs = partial(alt_term(c), p)
            partial(alt_term(c), p).abs <= c(Nat.0)
        }
    }
}

/// Any window of an alternating decreasing nonnegative series has absolute
/// value at most the first term of the window.
theorem alt_window_abs_bound(a: Nat -> Real, k: Nat, m: Nat) {
    is_decreasing_seq(a) and is_lower_bound(a, Real.0) and k <= m
    implies sum(map(k.until(m), alt_term(a))).abs <= a(k)
} by {
    if is_decreasing_seq(a) and is_lower_bound(a, Real.0) and k <= m {
        // The window is the partial sum of the tail of the alternating series.
        diff_partial(alt_term(a), k, m)
        sum(map(k.until(m), alt_term(a))) = partial(alt_term(a), m) - partial(alt_term(a), k)
        partial_tail_sub(alt_term(a), k, m - k)
        partial(tail(alt_term(a), k), m - k) = partial(alt_term(a), k + (m - k)) - partial(alt_term(a), k)
        add_sub(m, k)
        m - k + k = m
        partial(tail(alt_term(a), k), m - k) = partial(alt_term(a), m) - partial(alt_term(a), k)
        sum(map(k.until(m), alt_term(a))) = partial(tail(alt_term(a), k), m - k)
        // The tail of the alternating series is a signed alternating tail.
        alt_term_tail(a, k)
        tail(alt_term(a), k) = mul_seq(alternating_sign[Real](k), alt_term(tail(a, k)))
        partial_mul_seq_comm(alternating_sign[Real](k), alt_term(tail(a, k)))
        partial(mul_seq(alternating_sign[Real](k), alt_term(tail(a, k)))) = mul_seq(alternating_sign[Real](k), partial(alt_term(tail(a, k))))
        partial(tail(alt_term(a), k), m - k) = mul_seq(alternating_sign[Real](k), partial(alt_term(tail(a, k))))(m - k)
        mul_seq(alternating_sign[Real](k), partial(alt_term(tail(a, k))), m - k) = alternating_sign[Real](k) * partial(alt_term(tail(a, k)), m - k)
        sum(map(k.until(m), alt_term(a))) = alternating_sign[Real](k) * partial(alt_term(tail(a, k)), m - k)
        // Absolute values: |window| = |partial of the alternating tail|.
        mul_abs(alternating_sign[Real](k), partial(alt_term(tail(a, k)), m - k))
        (alternating_sign[Real](k) * partial(alt_term(tail(a, k)), m - k)).abs = alternating_sign[Real](k).abs * partial(alt_term(tail(a, k)), m - k).abs
        alt_sign_abs_one(k)
        alternating_sign[Real](k).abs = Real.1
        sum(map(k.until(m), alt_term(a))).abs = partial(alt_term(tail(a, k)), m - k).abs
        // The tail is decreasing and nonnegative, so the partial sum is bounded by tail(a, k)(0).
        tail_decreasing_seq(a, k)
        is_decreasing_seq(tail(a, k))
        tail_lower_bound_seq(a, k, Real.0)
        is_lower_bound(tail(a, k), Real.0)
        alt_partial_abs_bound(tail(a, k), m - k)
        partial(alt_term(tail(a, k)), m - k).abs <= tail(a, k)(Nat.0)
        tail_zero_at(a, k)
        tail(a, k)(Nat.0) = a(k)
        sum(map(k.until(m), alt_term(a))).abs <= a(k)
    }
}

/// An alternating decreasing nonnegative sequence whose terms vanish has an
/// alternating series satisfying the Cauchy criterion for series.
theorem alt_is_cauchy_series(a: Nat -> Real) {
    is_decreasing_seq(a) and is_lower_bound(a, Real.0) and converges_to(a, Real.0)
    implies is_cauchy_series(alt_term(a))
} by {
    if is_decreasing_seq(a) and is_lower_bound(a, Real.0) and converges_to(a, Real.0) {
        // The tail of a is eventually below any positive epsilon.
        converges_to(a, Real.0) = forall(e: Real) {
            e.is_positive implies exists(n: Nat) {
                tail_bound(a, Real.0, n, e)
            }
        }
        forall(eps: Real) {
            if eps.is_positive {
                exists(n: Nat) {
                    tail_bound(a, Real.0, n, eps)
                }
                let n0: Nat satisfy {
                    tail_bound(a, Real.0, n0, eps)
                }
                forall(k: Nat, m: Nat) {
                    if n0 <= k and k <= m {
                        tail_bound_implies_is_close(a, Real.0, n0, eps, k)
                        a(k).is_close(Real.0, eps)
                        (a(k) - Real.0).abs < eps
                        a(k) - Real.0 = a(k)
                        a(k).abs < eps
                        Real.0 <= a(k)
                        a(k).abs = a(k)
                        a(k) < eps
                        alt_window_abs_bound(a, k, m)
                        sum(map(k.until(m), alt_term(a))).abs <= a(k)
                        lte_lt_trans(sum(map(k.until(m), alt_term(a))).abs, a(k), eps)
                        sum(map(k.until(m), alt_term(a))).abs < eps
                    }
                }
                exists(k0: Nat, k1: Nat) {
                    n0 <= k0 and k0 <= k1 and not sum(map(k0.until(k1), alt_term(a))).abs < eps
                } or forall(k: Nat, m: Nat) {
                    n0 <= k and k <= m implies sum(map(k.until(m), alt_term(a))).abs < eps
                }
                forall(k: Nat, m: Nat) {
                    n0 <= k and k <= m implies sum(map(k.until(m), alt_term(a))).abs < eps
                }
                exists(n1: Nat) {
                    forall(k: Nat, m: Nat) {
                        n1 <= k and k <= m implies sum(map(k.until(m), alt_term(a))).abs < eps
                    }
                }
            }
        }
    }
}

/// The alternating series test (Leibniz): if a decreases to zero, then the
/// alternating series Σ (−1)ⁿ a(n) converges.
theorem alternating_series_test(a: Nat -> Real) {
    is_decreasing_seq(a) and vanishes(a) implies converges(partial(alt_term(a)))
} by {
    if is_decreasing_seq(a) and vanishes(a) {
        // The sequence is nonnegative since it decreases to zero.
        converges(a)
        limit(a) = Real.0
        decreasing_convergent_bounded_by_limit(a)
        is_lower_bound(a, limit(a))
        is_lower_bound(a, Real.0)
        // The tail of a is eventually below any positive epsilon.
        converges_imp_converges_to(a)
        converges_to(a, limit(a))
        converges_to(a, Real.0)
        // Use the Cauchy criterion for series.
        alt_is_cauchy_series(a)
        is_cauchy_series(alt_term(a))
        cauchy_series_imp_converges(alt_term(a))
        converges(partial(alt_term(a)))
    }
}

/// The ratio sequence |a(n+1)| / |a(n)| of a sequence.
define ratio_seq(a: Nat -> Real, n: Nat) -> Real {
    a(n.suc).abs / a(n).abs
}

/// A sequence converging to b is eventually strictly below any a with b < a.
theorem converges_to_strict_ub(q: Nat -> Real, b: Real, a: Real) {
    converges_to(q, b) and b < a implies exists(n: Nat) {
        forall(i: Nat) {
            n <= i implies q(i) < a
        }
    }
} by {
    if converges_to(q, b) and b < a {
        (a - b).is_positive
        converges_to(q, b) = forall(e: Real) {
            e.is_positive implies exists(n: Nat) {
                tail_bound(q, b, n, e)
            }
        }
        exists(n: Nat) {
            tail_bound(q, b, n, a - b)
        }
        let n: Nat satisfy {
            tail_bound(q, b, n, a - b)
        }
        forall(i: Nat) {
            if n <= i {
                tail_bound_implies_is_close(q, b, n, a - b, i)
                q(i).is_close(b, a - b)
                (q(i) - b).abs < a - b
                (q(i) - b) <= (q(i) - b).abs
                lte_lt_trans((q(i) - b), (q(i) - b).abs, a - b)
                q(i) - b < a - b
                lt_add_right(q(i) - b, a - b, b)
                (q(i) - b) + b < (a - b) + b
                (q(i) - b) + b = q(i)
                (a - b) + b = a
                q(i) < a
            }
        }
        exists(n0: Nat) {
            forall(i: Nat) {
                n0 <= i implies q(i) < a
            }
        }
    }
}

/// Between l and 1 there is a nonnegative real.
theorem exists_alpha_between(l: Real) {
    l < Real.1 implies exists(alpha: Real) {
        l < alpha and alpha < Real.1 and Real.0 <= alpha
    }
} by {
    if l < Real.1 {
        if l < Real.0 {
            Real.1.is_positive
            Real.0 < Real.1
            rat_between_reals(Real.0, Real.1)
            let r: Rat satisfy {
                Real.0 < Real.from_rat(r) and Real.from_rat(r) < Real.1
            }
            lt_trans(l, Real.0, Real.from_rat(r))
            l < Real.0
            Real.0 < Real.from_rat(r)
            l < Real.from_rat(r)
            Real.0 <= Real.from_rat(r)
            exists(alpha: Real) {
                l < alpha and alpha < Real.1 and Real.0 <= alpha
            }
        } else {
            Real.0 <= l
            rat_between_reals(l, Real.1)
            let r: Rat satisfy {
                l < Real.from_rat(r) and Real.from_rat(r) < Real.1
            }
            lte_lt_trans(Real.0, l, Real.from_rat(r))
            Real.0 < Real.from_rat(r)
            Real.0 <= Real.from_rat(r)
            exists(alpha: Real) {
                l < alpha and alpha < Real.1 and Real.0 <= alpha
            }
        }
    }
}

/// Ratio test, parameterized by the strict bound alpha: if the ratio sequence
/// converges to l < alpha < 1, then the series converges absolutely.
theorem ratio_test_abs_conv(a: Nat -> Real, l: Real, alpha: Real) {
    converges_to(ratio_seq(a), l)
    and l < alpha
    and alpha < Real.1
    and Real.0 <= alpha
    and always_nonzero(a)
    implies absolutely_converges(a)
} by {
    if converges_to(ratio_seq(a), l)
        and l < alpha
        and alpha < Real.1
        and Real.0 <= alpha
        and always_nonzero(a) {
        // The ratio sequence is eventually strictly below alpha.
        converges_to_strict_ub(ratio_seq(a), l, alpha)
        exists(n: Nat) {
            forall(i: Nat) {
                n <= i implies ratio_seq(a)(i) < alpha
            }
        }
        let big_n: Nat satisfy {
            forall(i: Nat) {
                big_n <= i implies ratio_seq(a)(i) < alpha
            }
        }
        // The ratio bound at the shift, in the defining form of
        // has_ratio_bound_eventually.
        forall(k: Nat) {
            big_n <= k and a(k) != Real.0 implies (a(k.suc).abs / a(k).abs) < alpha
        }
        forall(k: Nat) {
            k >= big_n and a(k) != Real.0 implies (a(k.suc).abs / a(k).abs) < alpha
        }
        has_ratio_bound_eventually(a, alpha)
        // Apply the eventual ratio test from Real.exp.ac.
        eventual_ratio_bound_implies_abs_conv(a, alpha)
        absolutely_converges(a)
    }
}

/// The ratio test: if |a(n+1)| / |a(n)| converges to l < 1, then the series
/// Σ a(n) converges absolutely.
theorem ratio_test(a: Nat -> Real, l: Real) {
    converges_to(ratio_seq(a), l) and l < Real.1 and always_nonzero(a)
    implies absolutely_converges(a)
} by {
    if converges_to(ratio_seq(a), l) and l < Real.1 and always_nonzero(a) {
        // Choose alpha with l < alpha < 1 and 0 <= alpha.
        exists_alpha_between(l)
        let alpha: Real satisfy {
            l < alpha and alpha < Real.1 and Real.0 <= alpha
        }
        ratio_test_abs_conv(a, l, alpha)
        absolutely_converges(a)
    }
}

// The root test: if the sequence of nth roots of |a(n)| converges to l < 1,
// then Σ a(n) converges absolutely.
//
// Proved in real.series_root_test (root_test). The root sequence there is
// |a(n)|^(1/(n+1)), built on the library's rpow machinery (abs_pow_value and
// the root-power identity (x^(1/p))^p = x from real.holder_minkowski); the
// proof follows the route sketched below.
//   (1) from convergence to l < alpha < 1, eventually |a(n)|^(1/(n+1)) < alpha;
//   (2) raising both sides to the (n+1)-st power gives |a(n)| <= alpha^(n+1),
//       using the root-power identity;
//   (3) the geometric majorant alpha^k dominates |a(k)| eventually, so
//       eventually_abs_le_geometric_absolutely_converges applies.

// The integral test: for a decreasing nonnegative f, the series Σ f(n)
// converges if and only if the integral of f converges.
//
// This needs the integral machinery (monotone comparison of the partial sums
// against step-function lower and upper integrals over [0, n]) together with
// the monotone convergence principle for the resulting bounds. It is too
// heavy for this file and is left as future work.
