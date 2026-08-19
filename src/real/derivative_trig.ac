/// Derivatives of sine and cosine.
///
/// Sine and cosine are defined by power series in `trig.ac`, with addition
/// formulas in `trig_identities.ac`.  Following the pattern of the exponential
/// derivative in `derivative_exp_log.ac`, we introduce the shifted series
///     x.sin / x = sum_n (-1)^n x^(2n) / (2n+1)!
/// and
///     (x.cos - 1) / x^2 = sum_n (-1)^(n+1) x^(2n) / (2n+2)!,
/// bound their remainders by powers of |x| (comparing against the geometric
/// series, which is enough for the small arguments used in the derivative
/// proofs), and then verify the derivative statements directly from the
/// epsilon-delta definition.
///
/// The main results are the derivative of sine being cosine
/// (`sin_has_derivative_at`, `sin_is_derivative_fn`) and the derivative of
/// cosine being minus sine (`cos_has_derivative_at`, `cos_is_derivative_fn`).
/// The key estimates are `sin_shift_sum_close_one` (the limit h.sin/h -> 1)
/// and `cos_shift_sum_bound` (the limit (h.cos - 1)/h -> 0).


from nat import Nat, pow_add, from_nat, pow_one, lte_one_factorial
from rat import Rat
from order import lte_trans, lt_trans, lt_of_lte_of_lt, lt_imp_lte
from list import partial
from data.basic.function_algebra import pointwise_neg
from algebra.ring.ring import alternating_sign, alternating_sign_zero
from real.real_field import Real
from real.real_ring import converges, limit, converges_to, mul_abs, lte_mul_nonneg_right, exists_small_mul_variant
from real.real_seq import converges_to_unique, add_seq, limit_add_seq, eps_smaller_than_both, convergent_converges_to_limit
from real.real_base import abs_neg, add_comm, add_assoc, abs_gte_zero, lte_abs, lte_lt_trans, lte_add_right
from real.abs_conv import absolutely_converges, abs_fn, absolutely_converges_imp_converges, absolutely_converges_comparison, abs_fn_tail_comm
from real.real_series import tail, tail_converges_to, const_converges, const_limit, const_seq, mul_seq, neg_seq, partial_mul_seq_comm, seq_lte, seq_lte_preserves_limit, partial_seq_lte, is_lower_bound, pow_nonneg, abs_pow, converges_mul_seq, add_seq_converges, neg_seq_converges, neg_seq_converges_to, partial_tail_sub, triangle_ineq, geom_converges
from real.limits import limit_abs_seq, abs_seq
from real.exp import factorial_pos, pow_suc, abs_div, two, mul_frac_assoc, two_positive, one_div_one_half_eq_two, one_div_two_eq_half
from real.trig import sin_term, cos_term, sin_term_abs_converges, cos_term_abs_converges, two_mul_suc, alternating_sign_abs, real_pow_mul_distrib
from real.trig_identities import sin_add, cos_add
from real.derivative_basic import has_derivative_at, difference_quotient, sub_ne_zero_of_ne
from real.derivative_rules import div_add_distrib
from real.derivative_exp_log import partial_abs_le_partial_abs_fn, half_pos, half_lt_one, geom_partial_bound, geom_partial_minus_one_bound
from real.harmonic import real_recip_antitone_pos, rat_from_nat_lte_of_nat_lte
from real.calculus_api import is_derivative_fn

numerals Real
numerals Nat

// General small lemmas about squares and geometric series.

/// A nonnegative real at most one has square at most itself.
theorem sq_lte_base(x: Real) {
    x >= Real.0 and x <= Real.1 implies x.pow(Nat.2) <= x
} by {
    if x >= Real.0 and x <= Real.1 {
        pow_suc(x, Nat.1)
        x.pow(Nat.1.suc) = x * x.pow(Nat.1)
        pow_one[Real](x)
        x.pow(Nat.1) = x
        Nat.1.suc = Nat.2
        x.pow(Nat.2) = x * x
        lte_mul_nonneg_right(x, Real.1, x)
        x * x <= Real.1 * x
        Real.1 * x = x
        x.pow(Nat.2) <= x
    }
}

/// A real with absolute value less than one has squared absolute value less than one.
theorem sq_abs_lt_one(x: Real) {
    x.abs < Real.1 implies x.abs.pow(Nat.2) < Real.1
} by {
    if x.abs < Real.1 {
        lt_imp_lte(x.abs, Real.1)
        x.abs <= Real.1
        abs_gte_zero(x)
        x.abs >= Real.0
        sq_lte_base(x.abs)
        x.abs.pow(Nat.2) <= x.abs
        lt_of_lte_of_lt(x.abs.pow(Nat.2), x.abs, Real.1)
        x.abs.pow(Nat.2) < Real.1
    }
}

/// The absolute value of a nonnegative real is itself.
theorem abs_of_nonneg(x: Real) {
    x >= Real.0 implies x.abs = x
} by {
    if x >= Real.0 {
        not x.is_negative
        x.abs = x
    }
}

/// The nth term of the shifted sine series: (-1)^n x^(2n) / (2n+1)!.
/// These are the terms of x.sin / x.
define sin_shift_term(x: Real, n: Nat) -> Real {
    alternating_sign[Real](n) * x.pow(Nat.2 * n) / Real.from_rat(Rat.from_nat((Nat.2 * n + Nat.1).factorial))
}

/// Multiplying a shifted sine term by x shifts it into the sine series.
/// x * ((-1)^n x^(2n) / (2n+1)!) = (-1)^n x^(2n+1) / (2n+1)!.
theorem sin_shift_term_mul_x(x: Real, n: Nat) {
    x * sin_shift_term(x, n) = sin_term(x, n)
} by {
    sin_shift_term(x, n) = alternating_sign[Real](n) * x.pow(Nat.2 * n) / Real.from_rat(Rat.from_nat((Nat.2 * n + Nat.1).factorial))
    x * sin_shift_term(x, n) = x * (alternating_sign[Real](n) * x.pow(Nat.2 * n) / Real.from_rat(Rat.from_nat((Nat.2 * n + Nat.1).factorial)))
    mul_frac_assoc(x, alternating_sign[Real](n) * x.pow(Nat.2 * n), Real.from_rat(Rat.from_nat((Nat.2 * n + Nat.1).factorial)))
    x * (alternating_sign[Real](n) * x.pow(Nat.2 * n) / Real.from_rat(Rat.from_nat((Nat.2 * n + Nat.1).factorial))) =
        (x * (alternating_sign[Real](n) * x.pow(Nat.2 * n))) / Real.from_rat(Rat.from_nat((Nat.2 * n + Nat.1).factorial))
    x * (alternating_sign[Real](n) * x.pow(Nat.2 * n)) = alternating_sign[Real](n) * (x * x.pow(Nat.2 * n))
    pow_suc(x, Nat.2 * n)
    x.pow((Nat.2 * n).suc) = x * x.pow(Nat.2 * n)
    Nat.2 * n + Nat.1 = (Nat.2 * n).suc
    x.pow(Nat.2 * n + Nat.1) = x * x.pow(Nat.2 * n)
    alternating_sign[Real](n) * (x * x.pow(Nat.2 * n)) = alternating_sign[Real](n) * x.pow(Nat.2 * n + Nat.1)
    x * (alternating_sign[Real](n) * x.pow(Nat.2 * n)) = alternating_sign[Real](n) * x.pow(Nat.2 * n + Nat.1)
    (x * (alternating_sign[Real](n) * x.pow(Nat.2 * n))) / Real.from_rat(Rat.from_nat((Nat.2 * n + Nat.1).factorial)) =
        alternating_sign[Real](n) * x.pow(Nat.2 * n + Nat.1) / Real.from_rat(Rat.from_nat((Nat.2 * n + Nat.1).factorial))
    x * sin_shift_term(x, n) = alternating_sign[Real](n) * x.pow(Nat.2 * n + Nat.1) / Real.from_rat(Rat.from_nat((Nat.2 * n + Nat.1).factorial))
    sin_term(x, n) = alternating_sign[Real](n) * x.pow(Nat.2 * n + Nat.1) / Real.from_rat(Rat.from_nat((Nat.2 * n + Nat.1).factorial))
    x * sin_shift_term(x, n) = sin_term(x, n)
}

/// The shifted sine terms are the sine terms after multiplication by x.
theorem mul_seq_sin_shift_eq_sin_term(x: Real) {
    mul_seq(x, sin_shift_term(x)) = sin_term(x)
} by {
    forall(n: Nat) {
        mul_seq(x, sin_shift_term(x), n) = x * sin_shift_term(x, n)
        sin_shift_term_mul_x(x, n)
        x * sin_shift_term(x, n) = sin_term(x, n)
        mul_seq(x, sin_shift_term(x), n) = sin_term(x, n)
        sin_term(x, n) = mul_seq(x, sin_shift_term(x), n)
    }
}

/// The absolute value of the nth shifted sine term is |x|^(2n) / (2n+1)!.
theorem sin_shift_term_abs(x: Real, n: Nat) {
    sin_shift_term(x, n).abs = x.abs.pow(Nat.2 * n) / Real.from_rat(Rat.from_nat((Nat.2 * n + Nat.1).factorial))
} by {
    let f = Real.from_rat(Rat.from_nat((Nat.2 * n + Nat.1).factorial))
    factorial_pos(Nat.2 * n + Nat.1)
    f > Real.0
    f != Real.0
    not f.is_negative
    f.abs = f

    sin_shift_term(x, n) = alternating_sign[Real](n) * x.pow(Nat.2 * n) / f
    sin_shift_term(x, n).abs = (alternating_sign[Real](n) * x.pow(Nat.2 * n) / f).abs
    abs_div(alternating_sign[Real](n) * x.pow(Nat.2 * n), f)
    (alternating_sign[Real](n) * x.pow(Nat.2 * n) / f).abs =
        (alternating_sign[Real](n) * x.pow(Nat.2 * n)).abs / f.abs
    mul_abs(alternating_sign[Real](n), x.pow(Nat.2 * n))
    (alternating_sign[Real](n) * x.pow(Nat.2 * n)).abs =
        alternating_sign[Real](n).abs * x.pow(Nat.2 * n).abs
    alternating_sign_abs(n)
    alternating_sign[Real](n).abs = Real.1
    abs_pow(x, Nat.2 * n)
    x.pow(Nat.2 * n).abs = x.abs.pow(Nat.2 * n)
    alternating_sign[Real](n).abs * x.pow(Nat.2 * n).abs =
        Real.1 * x.abs.pow(Nat.2 * n)
    Real.1 * x.abs.pow(Nat.2 * n) = x.abs.pow(Nat.2 * n)
    (alternating_sign[Real](n) * x.pow(Nat.2 * n)).abs = x.abs.pow(Nat.2 * n)
    (alternating_sign[Real](n) * x.pow(Nat.2 * n) / f).abs =
        x.abs.pow(Nat.2 * n) / f.abs
    x.abs.pow(Nat.2 * n) / f.abs = x.abs.pow(Nat.2 * n) / f
    sin_shift_term(x, n).abs = x.abs.pow(Nat.2 * n) / f
}

/// A shifted sine term is at most the corresponding power of the squared absolute value:
/// |x|^(2n) / (2n+1)! <= (|x|^2)^n.
theorem sin_shift_term_abs_le_geom(x: Real, n: Nat) {
    sin_shift_term(x, n).abs <= x.abs.pow(Nat.2).pow(n)
} by {
    sin_shift_term_abs(x, n)
    sin_shift_term(x, n).abs = x.abs.pow(Nat.2 * n) / Real.from_rat(Rat.from_nat((Nat.2 * n + Nat.1).factorial))
    let f = Real.from_rat(Rat.from_nat((Nat.2 * n + Nat.1).factorial))
    lte_one_factorial(Nat.2 * n + Nat.1)
    Nat.1 <= (Nat.2 * n + Nat.1).factorial
    rat_from_nat_lte_of_nat_lte(Nat.1, (Nat.2 * n + Nat.1).factorial)
    Rat.1 <= Rat.from_nat((Nat.2 * n + Nat.1).factorial)
    Real.from_rat(Rat.1) <= Real.from_rat(Rat.from_nat((Nat.2 * n + Nat.1).factorial))
    Real.from_rat(Rat.1) = Real.1
    Real.1 <= f
    f > Real.0
    real_recip_antitone_pos(Real.1, f)
    Real.1 / f <= Real.1 / Real.1
    Real.1 / Real.1 = Real.1
    Real.1 / f <= Real.1
    pow_nonneg(x.abs, Nat.2 * n)
    Real.0 <= x.abs.pow(Nat.2 * n)
    lte_mul_nonneg_right(Real.1 / f, Real.1, x.abs.pow(Nat.2 * n))
    (Real.1 / f) * x.abs.pow(Nat.2 * n) <= Real.1 * x.abs.pow(Nat.2 * n)
    x.abs.pow(Nat.2 * n) * (Real.1 / f) <= x.abs.pow(Nat.2 * n) * Real.1
    x.abs.pow(Nat.2 * n) / f <= x.abs.pow(Nat.2 * n)
    pow_suc(x.abs, Nat.1)
    x.abs.pow(Nat.1.suc) = x.abs * x.abs.pow(Nat.1)
    pow_one[Real](x.abs)
    x.abs.pow(Nat.1) = x.abs
    Nat.1.suc = Nat.2
    x.abs.pow(Nat.2) = x.abs * x.abs
    real_pow_mul_distrib(x.abs, x.abs, n)
    (x.abs * x.abs).pow(n) = x.abs.pow(n) * x.abs.pow(n)
    pow_add(x.abs, n, n)
    x.abs.pow(n + n) = x.abs.pow(n) * x.abs.pow(n)
    Nat.2 * n = n + n
    x.abs.pow(Nat.2 * n) = x.abs.pow(n) * x.abs.pow(n)
    (x.abs * x.abs).pow(n) = x.abs.pow(Nat.2 * n)
    x.abs.pow(Nat.2).pow(n) = x.abs.pow(Nat.2 * n)
    lte_trans(sin_shift_term(x, n).abs, x.abs.pow(Nat.2 * n) / f, x.abs.pow(Nat.2 * n))
    sin_shift_term(x, n).abs <= x.abs.pow(Nat.2 * n)
    sin_shift_term(x, n).abs <= x.abs.pow(Nat.2).pow(n)
}

/// The shifted sine series converges absolutely for small arguments.
theorem sin_shift_term_abs_converges_small(x: Real) {
    x.abs < Real.1 implies absolutely_converges(sin_shift_term(x))
} by {
    if x.abs < Real.1 {
        forall(n: Nat) {
            abs_gte_zero(x)
            Real.0 <= x.abs
            pow_nonneg(x.abs, Nat.2)
            Real.0 <= x.abs.pow(Nat.2)
            pow_nonneg(x.abs.pow(Nat.2), n)
            Real.0 <= x.abs.pow(Nat.2).pow(n)
        }
        is_lower_bound(x.abs.pow(Nat.2).pow, Real.0)
        forall(n: Nat) {
            abs_fn(sin_shift_term(x), n) = sin_shift_term(x, n).abs
            sin_shift_term_abs_le_geom(x, n)
            sin_shift_term(x, n).abs <= x.abs.pow(Nat.2).pow(n)
            abs_fn(sin_shift_term(x), n) <= x.abs.pow(Nat.2).pow(n)
        }
        seq_lte(abs_fn(sin_shift_term(x)), x.abs.pow(Nat.2).pow)
        sq_abs_lt_one(x)
        x.abs.pow(Nat.2) < Real.1
        abs_of_nonneg(x.abs.pow(Nat.2))
        x.abs.pow(Nat.2) >= Real.0
        x.abs.pow(Nat.2).abs = x.abs.pow(Nat.2)
        x.abs.pow(Nat.2).abs < Real.1
        geom_converges(x.abs.pow(Nat.2))
        converges(partial(x.abs.pow(Nat.2).pow))
        absolutely_converges_comparison(sin_shift_term(x), x.abs.pow(Nat.2).pow)
        absolutely_converges(sin_shift_term(x))
    }
}

/// The sum of the shifted sine series.
define sin_shift_sum(x: Real) -> Real {
    limit(partial(sin_shift_term(x)))
}

/// The shifted sine partial sums relate to the sine partial sums:
/// x * partial(sin_shift_term(x), n) = partial(sin_term(x), n).
theorem sin_shift_partial_mul(x: Real, n: Nat) {
    x * partial(sin_shift_term(x), n) = partial(sin_term(x), n)
} by {
    mul_seq_sin_shift_eq_sin_term(x)
    mul_seq(x, sin_shift_term(x)) = sin_term(x)
    partial(mul_seq(x, sin_shift_term(x))) = partial(sin_term(x))
    partial_mul_seq_comm(x, sin_shift_term(x))
    partial(mul_seq(x, sin_shift_term(x))) = mul_seq(x, partial(sin_shift_term(x)))
    partial(mul_seq(x, sin_shift_term(x)), n) = x * partial(sin_shift_term(x), n)
    partial(sin_term(x), n) = x * partial(sin_shift_term(x), n)
    x * partial(sin_shift_term(x), n) = partial(sin_term(x), n)
}

/// The sine satisfies x.sin = x * sum_{n>=0} (-1)^n x^(2n) / (2n+1)! for small x.
theorem sin_series_small(x: Real) {
    x.abs < Real.1 implies x.sin = x * sin_shift_sum(x)
} by {
    if x.abs < Real.1 {
        sin_shift_term_abs_converges_small(x)
        absolutely_converges(sin_shift_term(x))
        absolutely_converges_imp_converges(sin_shift_term(x))
        converges(partial(sin_shift_term(x)))
        converges_mul_seq(x, partial(sin_shift_term(x)))
        converges_to(mul_seq(x, partial(sin_shift_term(x))), x * limit(partial(sin_shift_term(x))))
        converges(mul_seq(x, partial(sin_shift_term(x))))
        convergent_converges_to_limit(mul_seq(x, partial(sin_shift_term(x))))
        converges_to(mul_seq(x, partial(sin_shift_term(x))), limit(mul_seq(x, partial(sin_shift_term(x)))))
        converges_to_unique(mul_seq(x, partial(sin_shift_term(x))), limit(mul_seq(x, partial(sin_shift_term(x)))),
            x * limit(partial(sin_shift_term(x))))
        limit(mul_seq(x, partial(sin_shift_term(x)))) = x * limit(partial(sin_shift_term(x)))
        forall(n: Nat) {
            sin_shift_partial_mul(x, n)
            x * partial(sin_shift_term(x), n) = partial(sin_term(x), n)
            mul_seq(x, partial(sin_shift_term(x)), n) = x * partial(sin_shift_term(x), n)
            partial(sin_term(x), n) = mul_seq(x, partial(sin_shift_term(x)), n)
            mul_seq(x, partial(sin_shift_term(x)), n) = partial(sin_term(x), n)
        }
        mul_seq(x, partial(sin_shift_term(x))) = partial(sin_term(x))
        limit(mul_seq(x, partial(sin_shift_term(x)))) = limit(partial(sin_term(x)))
        sin_term_abs_converges(x)
        absolutely_converges(sin_term(x))
        absolutely_converges_imp_converges(sin_term(x))
        converges(partial(sin_term(x)))
        convergent_converges_to_limit(partial(sin_term(x)))
        converges_to(partial(sin_term(x)), limit(partial(sin_term(x))))
        converges_to_unique(partial(sin_term(x)), limit(partial(sin_term(x))), x * limit(partial(sin_shift_term(x))))
        limit(partial(sin_term(x))) = x * limit(partial(sin_shift_term(x)))
        x.sin = limit(partial(sin_term(x)))
        sin_shift_sum(x) = limit(partial(sin_shift_term(x)))
        x.sin = x * sin_shift_sum(x)
    }
}

/// The nth term of the shifted cosine series: (-1)^(n+1) x^(2n) / (2n+2)!.
/// These are the terms of (x.cos - 1) / x^2.
define cos_shift_term(x: Real, n: Nat) -> Real {
    alternating_sign[Real](n.suc) * x.pow(Nat.2 * n) / Real.from_rat(Rat.from_nat((Nat.2 * n.suc).factorial))
}

/// Multiplying a shifted cosine term by x^2 shifts it into the cosine series.
/// x^2 * ((-1)^(n+1) x^(2n) / (2n+2)!) = (-1)^(n+1) x^(2n+2) / (2n+2)!.
theorem cos_shift_term_mul_x2(x: Real, n: Nat) {
    x.pow(Nat.2) * cos_shift_term(x, n) = cos_term(x, n.suc)
} by {
    cos_shift_term(x, n) = alternating_sign[Real](n.suc) * x.pow(Nat.2 * n) / Real.from_rat(Rat.from_nat((Nat.2 * n.suc).factorial))
    x.pow(Nat.2) * cos_shift_term(x, n) = x.pow(Nat.2) * (alternating_sign[Real](n.suc) * x.pow(Nat.2 * n) / Real.from_rat(Rat.from_nat((Nat.2 * n.suc).factorial)))
    mul_frac_assoc(x.pow(Nat.2), alternating_sign[Real](n.suc) * x.pow(Nat.2 * n), Real.from_rat(Rat.from_nat((Nat.2 * n.suc).factorial)))
    x.pow(Nat.2) * (alternating_sign[Real](n.suc) * x.pow(Nat.2 * n) / Real.from_rat(Rat.from_nat((Nat.2 * n.suc).factorial))) =
        (x.pow(Nat.2) * (alternating_sign[Real](n.suc) * x.pow(Nat.2 * n))) / Real.from_rat(Rat.from_nat((Nat.2 * n.suc).factorial))
    x.pow(Nat.2) * (alternating_sign[Real](n.suc) * x.pow(Nat.2 * n)) = alternating_sign[Real](n.suc) * (x.pow(Nat.2) * x.pow(Nat.2 * n))
    pow_add(x, Nat.2, Nat.2 * n)
    x.pow(Nat.2 + Nat.2 * n) = x.pow(Nat.2) * x.pow(Nat.2 * n)
    two_mul_suc(n)
    Nat.2 * n.suc = (Nat.2 * n).suc.suc
    Nat.2 + Nat.2 * n = Nat.2 * n.suc
    x.pow(Nat.2 * n.suc) = x.pow(Nat.2) * x.pow(Nat.2 * n)
    alternating_sign[Real](n.suc) * (x.pow(Nat.2) * x.pow(Nat.2 * n)) = alternating_sign[Real](n.suc) * x.pow(Nat.2 * n.suc)
    x.pow(Nat.2) * (alternating_sign[Real](n.suc) * x.pow(Nat.2 * n)) = alternating_sign[Real](n.suc) * x.pow(Nat.2 * n.suc)
    (x.pow(Nat.2) * (alternating_sign[Real](n.suc) * x.pow(Nat.2 * n))) / Real.from_rat(Rat.from_nat((Nat.2 * n.suc).factorial)) =
        alternating_sign[Real](n.suc) * x.pow(Nat.2 * n.suc) / Real.from_rat(Rat.from_nat((Nat.2 * n.suc).factorial))
    x.pow(Nat.2) * cos_shift_term(x, n) = alternating_sign[Real](n.suc) * x.pow(Nat.2 * n.suc) / Real.from_rat(Rat.from_nat((Nat.2 * n.suc).factorial))
    cos_term(x, n.suc) = alternating_sign[Real](n.suc) * x.pow(Nat.2 * n.suc) / Real.from_rat(Rat.from_nat((Nat.2 * n.suc).factorial))
    x.pow(Nat.2) * cos_shift_term(x, n) = cos_term(x, n.suc)
}

/// The shifted cosine terms are the shifted tail of the cosine series.
theorem mul_seq_cos_shift_eq_tail(x: Real) {
    mul_seq(x.pow(Nat.2), cos_shift_term(x)) = tail(cos_term(x), Nat.1)
} by {
    forall(n: Nat) {
        mul_seq(x.pow(Nat.2), cos_shift_term(x), n) = x.pow(Nat.2) * cos_shift_term(x, n)
        cos_shift_term_mul_x2(x, n)
        x.pow(Nat.2) * cos_shift_term(x, n) = cos_term(x, n.suc)
        tail(cos_term(x), Nat.1, n) = cos_term(x, Nat.1 + n)
        Nat.1 + n = n.suc
        tail(cos_term(x), Nat.1, n) = cos_term(x, n.suc)
        mul_seq(x.pow(Nat.2), cos_shift_term(x), n) = tail(cos_term(x), Nat.1, n)
        tail(cos_term(x), Nat.1, n) = mul_seq(x.pow(Nat.2), cos_shift_term(x), n)
    }
}

/// The absolute value of the nth shifted cosine term is |x|^(2n) / (2n+2)!.
theorem cos_shift_term_abs(x: Real, n: Nat) {
    cos_shift_term(x, n).abs = x.abs.pow(Nat.2 * n) / Real.from_rat(Rat.from_nat((Nat.2 * n.suc).factorial))
} by {
    let f = Real.from_rat(Rat.from_nat((Nat.2 * n.suc).factorial))
    factorial_pos(Nat.2 * n.suc)
    f > Real.0
    f != Real.0
    not f.is_negative
    f.abs = f

    cos_shift_term(x, n) = alternating_sign[Real](n.suc) * x.pow(Nat.2 * n) / f
    cos_shift_term(x, n).abs = (alternating_sign[Real](n.suc) * x.pow(Nat.2 * n) / f).abs
    abs_div(alternating_sign[Real](n.suc) * x.pow(Nat.2 * n), f)
    (alternating_sign[Real](n.suc) * x.pow(Nat.2 * n) / f).abs =
        (alternating_sign[Real](n.suc) * x.pow(Nat.2 * n)).abs / f.abs
    mul_abs(alternating_sign[Real](n.suc), x.pow(Nat.2 * n))
    (alternating_sign[Real](n.suc) * x.pow(Nat.2 * n)).abs =
        alternating_sign[Real](n.suc).abs * x.pow(Nat.2 * n).abs
    alternating_sign_abs(n.suc)
    alternating_sign[Real](n.suc).abs = Real.1
    abs_pow(x, Nat.2 * n)
    x.pow(Nat.2 * n).abs = x.abs.pow(Nat.2 * n)
    alternating_sign[Real](n.suc).abs * x.pow(Nat.2 * n).abs =
        Real.1 * x.abs.pow(Nat.2 * n)
    Real.1 * x.abs.pow(Nat.2 * n) = x.abs.pow(Nat.2 * n)
    (alternating_sign[Real](n.suc) * x.pow(Nat.2 * n)).abs = x.abs.pow(Nat.2 * n)
    (alternating_sign[Real](n.suc) * x.pow(Nat.2 * n) / f).abs =
        x.abs.pow(Nat.2 * n) / f.abs
    x.abs.pow(Nat.2 * n) / f.abs = x.abs.pow(Nat.2 * n) / f
    cos_shift_term(x, n).abs = x.abs.pow(Nat.2 * n) / f
}

/// A shifted cosine term is at most the corresponding power of the squared absolute value:
/// |x|^(2n) / (2n+2)! <= (|x|^2)^n.
theorem cos_shift_term_abs_le_geom(x: Real, n: Nat) {
    cos_shift_term(x, n).abs <= x.abs.pow(Nat.2).pow(n)
} by {
    cos_shift_term_abs(x, n)
    cos_shift_term(x, n).abs = x.abs.pow(Nat.2 * n) / Real.from_rat(Rat.from_nat((Nat.2 * n.suc).factorial))
    let f = Real.from_rat(Rat.from_nat((Nat.2 * n.suc).factorial))
    lte_one_factorial(Nat.2 * n.suc)
    Nat.1 <= (Nat.2 * n.suc).factorial
    rat_from_nat_lte_of_nat_lte(Nat.1, (Nat.2 * n.suc).factorial)
    Rat.1 <= Rat.from_nat((Nat.2 * n.suc).factorial)
    Real.from_rat(Rat.1) <= Real.from_rat(Rat.from_nat((Nat.2 * n.suc).factorial))
    Real.from_rat(Rat.1) = Real.1
    Real.1 <= f
    f > Real.0
    real_recip_antitone_pos(Real.1, f)
    Real.1 / f <= Real.1 / Real.1
    Real.1 / Real.1 = Real.1
    Real.1 / f <= Real.1
    pow_nonneg(x.abs, Nat.2 * n)
    Real.0 <= x.abs.pow(Nat.2 * n)
    lte_mul_nonneg_right(Real.1 / f, Real.1, x.abs.pow(Nat.2 * n))
    (Real.1 / f) * x.abs.pow(Nat.2 * n) <= Real.1 * x.abs.pow(Nat.2 * n)
    x.abs.pow(Nat.2 * n) * (Real.1 / f) <= x.abs.pow(Nat.2 * n) * Real.1
    x.abs.pow(Nat.2 * n) / f <= x.abs.pow(Nat.2 * n)
    pow_suc(x.abs, Nat.1)
    x.abs.pow(Nat.1.suc) = x.abs * x.abs.pow(Nat.1)
    pow_one[Real](x.abs)
    x.abs.pow(Nat.1) = x.abs
    Nat.1.suc = Nat.2
    x.abs.pow(Nat.2) = x.abs * x.abs
    real_pow_mul_distrib(x.abs, x.abs, n)
    (x.abs * x.abs).pow(n) = x.abs.pow(n) * x.abs.pow(n)
    pow_add(x.abs, n, n)
    x.abs.pow(n + n) = x.abs.pow(n) * x.abs.pow(n)
    Nat.2 * n = n + n
    x.abs.pow(Nat.2 * n) = x.abs.pow(n) * x.abs.pow(n)
    (x.abs * x.abs).pow(n) = x.abs.pow(Nat.2 * n)
    x.abs.pow(Nat.2).pow(n) = x.abs.pow(Nat.2 * n)
    lte_trans(cos_shift_term(x, n).abs, x.abs.pow(Nat.2 * n) / f, x.abs.pow(Nat.2 * n))
    cos_shift_term(x, n).abs <= x.abs.pow(Nat.2 * n)
    cos_shift_term(x, n).abs <= x.abs.pow(Nat.2).pow(n)
}

/// The shifted cosine series converges absolutely for small arguments.
theorem cos_shift_term_abs_converges_small(x: Real) {
    x.abs < Real.1 implies absolutely_converges(cos_shift_term(x))
} by {
    if x.abs < Real.1 {
        forall(n: Nat) {
            abs_gte_zero(x)
            Real.0 <= x.abs
            pow_nonneg(x.abs, Nat.2)
            Real.0 <= x.abs.pow(Nat.2)
            pow_nonneg(x.abs.pow(Nat.2), n)
            Real.0 <= x.abs.pow(Nat.2).pow(n)
        }
        is_lower_bound(x.abs.pow(Nat.2).pow, Real.0)
        forall(n: Nat) {
            abs_fn(cos_shift_term(x), n) = cos_shift_term(x, n).abs
            cos_shift_term_abs_le_geom(x, n)
            cos_shift_term(x, n).abs <= x.abs.pow(Nat.2).pow(n)
            abs_fn(cos_shift_term(x), n) <= x.abs.pow(Nat.2).pow(n)
        }
        seq_lte(abs_fn(cos_shift_term(x)), x.abs.pow(Nat.2).pow)
        sq_abs_lt_one(x)
        x.abs.pow(Nat.2) < Real.1
        abs_of_nonneg(x.abs.pow(Nat.2))
        x.abs.pow(Nat.2) >= Real.0
        x.abs.pow(Nat.2).abs = x.abs.pow(Nat.2)
        x.abs.pow(Nat.2).abs < Real.1
        geom_converges(x.abs.pow(Nat.2))
        converges(partial(x.abs.pow(Nat.2).pow))
        absolutely_converges_comparison(cos_shift_term(x), x.abs.pow(Nat.2).pow)
        absolutely_converges(cos_shift_term(x))
    }
}

/// The sum of the shifted cosine series.
define cos_shift_sum(x: Real) -> Real {
    limit(partial(cos_shift_term(x)))
}

/// The shifted cosine partial sums relate to the cosine partial sums:
/// x^2 * partial(cos_shift_term(x), n) = partial(cos_term(x), n+1) - 1.
theorem cos_shift_partial_mul(x: Real, n: Nat) {
    x.pow(Nat.2) * partial(cos_shift_term(x), n) = partial(cos_term(x), n.suc) - Real.1
} by {
    mul_seq_cos_shift_eq_tail(x)
    mul_seq(x.pow(Nat.2), cos_shift_term(x)) = tail(cos_term(x), Nat.1)
    partial(mul_seq(x.pow(Nat.2), cos_shift_term(x))) = partial(tail(cos_term(x), Nat.1))
    partial_mul_seq_comm(x.pow(Nat.2), cos_shift_term(x))
    partial(mul_seq(x.pow(Nat.2), cos_shift_term(x))) = mul_seq(x.pow(Nat.2), partial(cos_shift_term(x)))
    partial(mul_seq(x.pow(Nat.2), cos_shift_term(x)), n) = x.pow(Nat.2) * partial(cos_shift_term(x), n)
    partial_tail_sub(cos_term(x), Nat.1, n)
    partial(tail(cos_term(x), Nat.1), n) =
        partial(cos_term(x), Nat.1 + n) - partial(cos_term(x), Nat.1)
    Nat.1 + n = n.suc
    partial(cos_term(x), Nat.1 + n) = partial(cos_term(x), n.suc)
    partial(cos_term(x), Nat.1) = cos_term(x, Nat.0)
    cos_term(x, Nat.0) = alternating_sign[Real](Nat.0) * x.pow(Nat.0) / Real.from_rat(Rat.from_nat(Nat.0.factorial))
    alternating_sign_zero[Real]
    alternating_sign[Real](Nat.0) = Real.1
    x.pow(Nat.0) = Real.1
    Nat.0.factorial = Nat.1
    Real.from_rat(Rat.from_nat(Nat.1)) = Real.1
    Real.1 * Real.1 = Real.1
    Real.1 / Real.1 = Real.1
    cos_term(x, Nat.0) = Real.1
    partial(cos_term(x), Nat.1) = Real.1
    partial(tail(cos_term(x), Nat.1), n) = partial(cos_term(x), n.suc) - Real.1
    x.pow(Nat.2) * partial(cos_shift_term(x), n) = partial(cos_term(x), n.suc) - Real.1
}

/// The cosine satisfies x.cos - 1 = x^2 * sum_{n>=0} (-1)^(n+1) x^(2n) / (2n+2)! for small x.
theorem cos_sub_one_series_small(x: Real) {
    x.abs < Real.1 implies x.cos - Real.1 = x.pow(Nat.2) * cos_shift_sum(x)
} by {
    if x.abs < Real.1 {
        cos_shift_term_abs_converges_small(x)
        absolutely_converges(cos_shift_term(x))
        absolutely_converges_imp_converges(cos_shift_term(x))
        converges(partial(cos_shift_term(x)))
        converges_mul_seq(x.pow(Nat.2), partial(cos_shift_term(x)))
        converges_to(mul_seq(x.pow(Nat.2), partial(cos_shift_term(x))), x.pow(Nat.2) * limit(partial(cos_shift_term(x))))
        converges(mul_seq(x.pow(Nat.2), partial(cos_shift_term(x))))
        convergent_converges_to_limit(mul_seq(x.pow(Nat.2), partial(cos_shift_term(x))))
        converges_to(mul_seq(x.pow(Nat.2), partial(cos_shift_term(x))), limit(mul_seq(x.pow(Nat.2), partial(cos_shift_term(x)))))
        converges_to_unique(mul_seq(x.pow(Nat.2), partial(cos_shift_term(x))), limit(mul_seq(x.pow(Nat.2), partial(cos_shift_term(x)))),
            x.pow(Nat.2) * limit(partial(cos_shift_term(x))))
        limit(mul_seq(x.pow(Nat.2), partial(cos_shift_term(x)))) = x.pow(Nat.2) * limit(partial(cos_shift_term(x)))

        forall(n: Nat) {
            cos_shift_partial_mul(x, n)
            x.pow(Nat.2) * partial(cos_shift_term(x), n) = partial(cos_term(x), n.suc) - Real.1
            mul_seq(x.pow(Nat.2), partial(cos_shift_term(x)), n) = x.pow(Nat.2) * partial(cos_shift_term(x), n)
            add_seq(tail(partial(cos_term(x)), Nat.1), neg_seq(const_seq(Real.1)), n) =
                tail(partial(cos_term(x)), Nat.1, n) + neg_seq(const_seq(Real.1), n)
            neg_seq(const_seq(Real.1), n) = mul_seq(-Real.1, const_seq(Real.1), n)
            mul_seq(-Real.1, const_seq(Real.1), n) = -Real.1 * const_seq(Real.1, n)
            const_seq(Real.1, n) = Real.1
            -Real.1 * Real.1 = -Real.1
            tail(partial(cos_term(x)), Nat.1, n) + neg_seq(const_seq(Real.1), n) =
                tail(partial(cos_term(x)), Nat.1, n) - Real.1
            add_seq(tail(partial(cos_term(x)), Nat.1), neg_seq(const_seq(Real.1)), n) =
                tail(partial(cos_term(x)), Nat.1, n) - Real.1
            tail(partial(cos_term(x)), Nat.1, n) = partial(cos_term(x), Nat.1 + n)
            Nat.1 + n = n.suc
            tail(partial(cos_term(x)), Nat.1, n) = partial(cos_term(x), n.suc)
            tail(partial(cos_term(x)), Nat.1, n) - Real.1 = partial(cos_term(x), n.suc) - Real.1
            mul_seq(x.pow(Nat.2), partial(cos_shift_term(x)), n) =
                add_seq(tail(partial(cos_term(x)), Nat.1), neg_seq(const_seq(Real.1)), n)
            add_seq(tail(partial(cos_term(x)), Nat.1), neg_seq(const_seq(Real.1)), n) =
                mul_seq(x.pow(Nat.2), partial(cos_shift_term(x)), n)
        }
        mul_seq(x.pow(Nat.2), partial(cos_shift_term(x))) =
            add_seq(tail(partial(cos_term(x)), Nat.1), neg_seq(const_seq(Real.1)))
        limit(mul_seq(x.pow(Nat.2), partial(cos_shift_term(x)))) =
            limit(add_seq(tail(partial(cos_term(x)), Nat.1), neg_seq(const_seq(Real.1))))

        cos_term_abs_converges(x)
        absolutely_converges(cos_term(x))
        absolutely_converges_imp_converges(cos_term(x))
        converges(partial(cos_term(x)))
        tail_converges_to(partial(cos_term(x)), Nat.1)
        converges_to(tail(partial(cos_term(x)), Nat.1), limit(partial(cos_term(x))))
        converges(tail(partial(cos_term(x)), Nat.1))
        const_converges(Real.1)
        converges(const_seq(Real.1))
        neg_seq_converges(const_seq(Real.1))
        converges(neg_seq(const_seq(Real.1)))
        limit_add_seq(tail(partial(cos_term(x)), Nat.1), neg_seq(const_seq(Real.1)))
        add_seq_converges(tail(partial(cos_term(x)), Nat.1), neg_seq(const_seq(Real.1)))
        converges(add_seq(tail(partial(cos_term(x)), Nat.1), neg_seq(const_seq(Real.1))))
        convergent_converges_to_limit(add_seq(tail(partial(cos_term(x)), Nat.1), neg_seq(const_seq(Real.1))))
        converges_to(add_seq(tail(partial(cos_term(x)), Nat.1), neg_seq(const_seq(Real.1))),
            limit(add_seq(tail(partial(cos_term(x)), Nat.1), neg_seq(const_seq(Real.1)))))
        converges_to_unique(add_seq(tail(partial(cos_term(x)), Nat.1), neg_seq(const_seq(Real.1))),
            limit(add_seq(tail(partial(cos_term(x)), Nat.1), neg_seq(const_seq(Real.1)))),
            limit(tail(partial(cos_term(x)), Nat.1)) + limit(neg_seq(const_seq(Real.1))))
        limit(add_seq(tail(partial(cos_term(x)), Nat.1), neg_seq(const_seq(Real.1)))) =
            limit(tail(partial(cos_term(x)), Nat.1)) + limit(neg_seq(const_seq(Real.1)))
        convergent_converges_to_limit(tail(partial(cos_term(x)), Nat.1))
        converges_to(tail(partial(cos_term(x)), Nat.1), limit(tail(partial(cos_term(x)), Nat.1)))
        converges_to_unique(tail(partial(cos_term(x)), Nat.1), limit(tail(partial(cos_term(x)), Nat.1)), limit(partial(cos_term(x))))
        limit(tail(partial(cos_term(x)), Nat.1)) = limit(partial(cos_term(x)))
        neg_seq_converges_to(const_seq(Real.1))
        converges_to(neg_seq(const_seq(Real.1)), -limit(const_seq(Real.1)))
        const_limit(Real.1)
        limit(const_seq(Real.1)) = Real.1
        limit(neg_seq(const_seq(Real.1))) = -Real.1
        limit(add_seq(tail(partial(cos_term(x)), Nat.1), neg_seq(const_seq(Real.1)))) =
            limit(partial(cos_term(x))) - Real.1
        limit(mul_seq(x.pow(Nat.2), partial(cos_shift_term(x)))) =
            limit(partial(cos_term(x))) - Real.1
        x.pow(Nat.2) * limit(partial(cos_shift_term(x))) =
            limit(partial(cos_term(x))) - Real.1
        x.cos = limit(partial(cos_term(x)))
        x.pow(Nat.2) * limit(partial(cos_shift_term(x))) = x.cos - Real.1
        cos_shift_sum(x) = limit(partial(cos_shift_term(x)))
        x.pow(Nat.2) * cos_shift_sum(x) = x.cos - Real.1
        x.cos - Real.1 = x.pow(Nat.2) * cos_shift_sum(x)
    }
}

// Bounds for the shifted sums.

/// The tail of the shifted sine series from index one is bounded by twice the squared argument.
theorem sin_shift_tail_partial_bound(h: Real, n: Nat) {
    h.abs < Real.1 / two implies partial(tail(abs_fn(sin_shift_term(h)), Nat.1), n) <= two * h.abs.pow(Nat.2)
} by {
    if h.abs < Real.1 / two {
        forall(j: Nat) {
            tail(abs_fn(sin_shift_term(h)), Nat.1, j) = abs_fn(sin_shift_term(h))(Nat.1 + j)
            abs_fn(sin_shift_term(h))(Nat.1 + j) = sin_shift_term(h, Nat.1 + j).abs
            Nat.1 + j = j.suc
            abs_fn(sin_shift_term(h))(Nat.1 + j) = sin_shift_term(h, j.suc).abs
            sin_shift_term_abs_le_geom(h, j.suc)
            sin_shift_term(h, j.suc).abs <= h.abs.pow(Nat.2).pow(j.suc)
            h.abs.pow(Nat.2).pow(j.suc) = h.abs.pow(Nat.2).pow(j) * h.abs.pow(Nat.2)
            h.abs.pow(Nat.2).pow(j.suc) = h.abs.pow(Nat.2) * h.abs.pow(Nat.2).pow(j)
            tail(abs_fn(sin_shift_term(h)), Nat.1, j) <= h.abs.pow(Nat.2) * h.abs.pow(Nat.2).pow(j)
            tail(h.abs.pow(Nat.2).pow, Nat.1, j) = h.abs.pow(Nat.2).pow(Nat.1 + j)
            h.abs.pow(Nat.2).pow(Nat.1 + j) = h.abs.pow(Nat.2).pow(j.suc)
            tail(h.abs.pow(Nat.2).pow, Nat.1, j) = h.abs.pow(Nat.2) * h.abs.pow(Nat.2).pow(j)
            tail(abs_fn(sin_shift_term(h)), Nat.1, j) <= tail(h.abs.pow(Nat.2).pow, Nat.1, j)
        }
        seq_lte(tail(abs_fn(sin_shift_term(h)), Nat.1), tail(h.abs.pow(Nat.2).pow, Nat.1))
        partial_seq_lte(tail(abs_fn(sin_shift_term(h)), Nat.1), tail(h.abs.pow(Nat.2).pow, Nat.1))
        seq_lte(partial(tail(abs_fn(sin_shift_term(h)), Nat.1)), partial(tail(h.abs.pow(Nat.2).pow, Nat.1)))
        partial(tail(abs_fn(sin_shift_term(h)), Nat.1), n) <= partial(tail(h.abs.pow(Nat.2).pow, Nat.1), n)
        partial_tail_sub(h.abs.pow(Nat.2).pow, Nat.1, n)
        partial(tail(h.abs.pow(Nat.2).pow, Nat.1), n) =
            partial(h.abs.pow(Nat.2).pow, Nat.1 + n) - partial(h.abs.pow(Nat.2).pow, Nat.1)
        partial(h.abs.pow(Nat.2).pow, Nat.1) = h.abs.pow(Nat.2).pow(Nat.0)
        h.abs.pow(Nat.2).pow(Nat.0) = Real.1
        partial(h.abs.pow(Nat.2).pow, Nat.1) = Real.1
        Nat.1 + n = n.suc
        partial(tail(h.abs.pow(Nat.2).pow, Nat.1), n) =
            partial(h.abs.pow(Nat.2).pow, n.suc) - Real.1
        abs_gte_zero(h)
        Real.0 <= h.abs
        lt_imp_lte(h.abs, Real.1 / two)
        h.abs <= Real.1 / two
        half_lt_one
        Real.1 / two < Real.1
        lt_of_lte_of_lt(h.abs, Real.1 / two, Real.1)
        h.abs < Real.1
        sq_lte_base(h.abs)
        h.abs.pow(Nat.2) <= h.abs
        lt_of_lte_of_lt(h.abs.pow(Nat.2), h.abs, Real.1)
        h.abs.pow(Nat.2) < Real.1
        abs_of_nonneg(h.abs.pow(Nat.2))
        pow_nonneg(h.abs, Nat.2)
        h.abs.pow(Nat.2) >= Real.0
        h.abs.pow(Nat.2).abs = h.abs.pow(Nat.2)
        h.abs.pow(Nat.2).abs < Real.1
        geom_partial_minus_one_bound(h.abs.pow(Nat.2), n)
        partial(h.abs.pow(Nat.2).pow, n.suc) - Real.1 <= h.abs.pow(Nat.2) / (Real.1 - h.abs.pow(Nat.2))
        partial(tail(h.abs.pow(Nat.2).pow, Nat.1), n) <= h.abs.pow(Nat.2) / (Real.1 - h.abs.pow(Nat.2))
        lte_trans(partial(tail(abs_fn(sin_shift_term(h)), Nat.1), n),
            partial(tail(h.abs.pow(Nat.2).pow, Nat.1), n), h.abs.pow(Nat.2) / (Real.1 - h.abs.pow(Nat.2)))
        partial(tail(abs_fn(sin_shift_term(h)), Nat.1), n) <= h.abs.pow(Nat.2) / (Real.1 - h.abs.pow(Nat.2))
        lt_of_lte_of_lt(h.abs.pow(Nat.2), h.abs, Real.1 / two)
        h.abs.pow(Nat.2) < Real.1 / two
        lte_add_right(h.abs.pow(Nat.2), Real.1 / two, -Real.1)
        h.abs.pow(Nat.2) + -Real.1 < Real.1 / two + -Real.1
        h.abs.pow(Nat.2) - Real.1 < Real.1 / two - Real.1
        -(h.abs.pow(Nat.2) - Real.1) > -(Real.1 / two - Real.1)
        Real.1 - h.abs.pow(Nat.2) > Real.1 - Real.1 / two
        one_div_two_eq_half
        Real.1 / two = Real.one_half
        Real.1 - Real.1 / two = Real.1 - Real.one_half
        Real.1 - Real.one_half = Real.one_half
        Real.1 - Real.one_half = Real.1 / two
        Real.1 - Real.1 / two = Real.1 / two
Real.1 - h.abs.pow(Nat.2) > Real.1 / two
        lt_imp_lte(Real.1 / two, Real.1 - h.abs.pow(Nat.2))
        Real.1 / two <= Real.1 - h.abs.pow(Nat.2)
        half_pos
        Real.1 / two > Real.0
        Real.1 - h.abs.pow(Nat.2) > Real.0
        real_recip_antitone_pos(Real.1 / two, Real.1 - h.abs.pow(Nat.2))
        Real.1 / (Real.1 - h.abs.pow(Nat.2)) <= Real.1 / (Real.1 / two)
        one_div_one_half_eq_two
        Real.1 / Real.one_half = two
        Real.1 / two = Real.one_half
        Real.1 / (Real.1 / two) = two
        Real.1 / (Real.1 - h.abs.pow(Nat.2)) <= two
        lte_mul_nonneg_right(Real.1 / (Real.1 - h.abs.pow(Nat.2)), two, h.abs.pow(Nat.2))
        (Real.1 / (Real.1 - h.abs.pow(Nat.2))) * h.abs.pow(Nat.2) <= two * h.abs.pow(Nat.2)
        h.abs.pow(Nat.2) * (Real.1 / (Real.1 - h.abs.pow(Nat.2))) <= two * h.abs.pow(Nat.2)
        h.abs.pow(Nat.2) / (Real.1 - h.abs.pow(Nat.2)) <= two * h.abs.pow(Nat.2)
        lte_trans(partial(tail(abs_fn(sin_shift_term(h)), Nat.1), n),
            h.abs.pow(Nat.2) / (Real.1 - h.abs.pow(Nat.2)), two * h.abs.pow(Nat.2))
        partial(tail(abs_fn(sin_shift_term(h)), Nat.1), n) <= two * h.abs.pow(Nat.2)
    }
}

/// The shifted sine sum is close to one: |S(h) - 1| <= 2|h|^2 when |h| <= 1/2.
theorem sin_shift_sum_close_one(h: Real) {
    h.abs < Real.1 / two implies (sin_shift_sum(h) - Real.1).abs <= two * h.abs.pow(Nat.2)
} by {
    if h.abs < Real.1 / two {
        lt_imp_lte(h.abs, Real.1 / two)
        h.abs <= Real.1 / two
        half_lt_one
        Real.1 / two < Real.1
        lt_of_lte_of_lt(h.abs, Real.1 / two, Real.1)
        h.abs < Real.1
        sin_shift_term_abs_converges_small(h)
        absolutely_converges(sin_shift_term(h))
        absolutely_converges_imp_converges(sin_shift_term(h))
        converges(partial(sin_shift_term(h)))
        define a(n: Nat) -> Real {
            partial(sin_shift_term(h), n) - Real.1
        }
        forall(n: Nat) {
            tail(a, Nat.1, n) = a(Nat.1 + n)
            a(Nat.1 + n) = partial(sin_shift_term(h), Nat.1 + n) - Real.1
            Nat.1 + n = n.suc
            a(Nat.1 + n) = partial(sin_shift_term(h), n.suc) - Real.1
            partial(tail(sin_shift_term(h), Nat.1), n) =
                partial(sin_shift_term(h), Nat.1 + n) - partial(sin_shift_term(h), Nat.1)
            partial(sin_shift_term(h), Nat.1) = sin_shift_term(h, Nat.0)
            sin_shift_term(h, Nat.0) = alternating_sign[Real](Nat.0) * h.pow(Nat.0) / Real.from_rat(Rat.from_nat((Nat.2 * Nat.0 + Nat.1).factorial))
            alternating_sign_zero[Real]
            alternating_sign[Real](Nat.0) = Real.1
            h.pow(Nat.0) = Real.1
            Nat.2 * Nat.0 + Nat.1 = Nat.1
            Nat.1.factorial = Nat.1
            Real.from_rat(Rat.from_nat(Nat.1)) = Real.1
            Real.1 * Real.1 = Real.1
            Real.1 / Real.1 = Real.1
            sin_shift_term(h, Nat.0) = Real.1
            partial(sin_shift_term(h), Nat.1) = Real.1
            partial(tail(sin_shift_term(h), Nat.1), n) =
                partial(sin_shift_term(h), n.suc) - Real.1
            tail(a, Nat.1, n) = partial(tail(sin_shift_term(h), Nat.1), n)
            partial_abs_le_partial_abs_fn(tail(sin_shift_term(h), Nat.1), n)
            partial(tail(sin_shift_term(h), Nat.1), n).abs <= partial(abs_fn(tail(sin_shift_term(h), Nat.1)), n)
            abs_fn_tail_comm(sin_shift_term(h), Nat.1)
            abs_fn(tail(sin_shift_term(h), Nat.1)) = tail(abs_fn(sin_shift_term(h)), Nat.1)
            partial(abs_fn(tail(sin_shift_term(h), Nat.1)), n) =
                partial(tail(abs_fn(sin_shift_term(h)), Nat.1), n)
            partial(tail(sin_shift_term(h), Nat.1), n).abs <= partial(tail(abs_fn(sin_shift_term(h)), Nat.1), n)
            tail(a, Nat.1, n).abs = partial(tail(sin_shift_term(h), Nat.1), n).abs
            tail(a, Nat.1, n).abs <= partial(tail(abs_fn(sin_shift_term(h)), Nat.1), n)
            sin_shift_tail_partial_bound(h, n)
            partial(tail(abs_fn(sin_shift_term(h)), Nat.1), n) <= two * h.abs.pow(Nat.2)
            lte_trans(tail(a, Nat.1, n).abs, partial(tail(abs_fn(sin_shift_term(h)), Nat.1), n), two * h.abs.pow(Nat.2))
            tail(a, Nat.1, n).abs <= two * h.abs.pow(Nat.2)
        }
        abs_gte_zero(h)
        Real.0 <= h.abs
        pow_nonneg(h.abs, Nat.2)
        Real.0 <= h.abs.pow(Nat.2)
        const_converges(two * h.abs.pow(Nat.2))
        converges(const_seq(two * h.abs.pow(Nat.2)))
        forall(n: Nat) {
            abs_seq(tail(a, Nat.1))(n) = tail(a, Nat.1, n).abs
            tail(a, Nat.1, n).abs <= two * h.abs.pow(Nat.2)
            const_seq(two * h.abs.pow(Nat.2), n) = two * h.abs.pow(Nat.2)
            abs_seq(tail(a, Nat.1))(n) <= const_seq(two * h.abs.pow(Nat.2), n)
        }
        seq_lte(abs_seq(tail(a, Nat.1)), const_seq(two * h.abs.pow(Nat.2)))
        add_seq_converges(partial(sin_shift_term(h)), neg_seq(const_seq(Real.1)))
        converges(add_seq(partial(sin_shift_term(h)), neg_seq(const_seq(Real.1))))
        forall(n: Nat) {
            add_seq(partial(sin_shift_term(h)), neg_seq(const_seq(Real.1)), n) =
                partial(sin_shift_term(h), n) + neg_seq(const_seq(Real.1), n)
            neg_seq(const_seq(Real.1), n) = mul_seq(-Real.1, const_seq(Real.1), n)
            mul_seq(-Real.1, const_seq(Real.1), n) = -Real.1 * const_seq(Real.1, n)
            const_seq(Real.1, n) = Real.1
            -Real.1 * Real.1 = -Real.1
            add_seq(partial(sin_shift_term(h)), neg_seq(const_seq(Real.1)), n) =
                partial(sin_shift_term(h), n) - Real.1
            add_seq(partial(sin_shift_term(h)), neg_seq(const_seq(Real.1)), n) = a(n)
        }
        add_seq(partial(sin_shift_term(h)), neg_seq(const_seq(Real.1))) = a
        converges(a)
        tail_converges_to(a, Nat.1)
        converges_to(tail(a, Nat.1), limit(a))
        converges(tail(a, Nat.1))
        limit_abs_seq(tail(a, Nat.1))
        converges_to(abs_seq(tail(a, Nat.1)), limit(tail(a, Nat.1)).abs)
        converges(abs_seq(tail(a, Nat.1)))
        convergent_converges_to_limit(abs_seq(tail(a, Nat.1)))
        converges_to(abs_seq(tail(a, Nat.1)), limit(abs_seq(tail(a, Nat.1))))
        converges_to_unique(abs_seq(tail(a, Nat.1)), limit(abs_seq(tail(a, Nat.1))),
            limit(tail(a, Nat.1)).abs)
        limit(abs_seq(tail(a, Nat.1))) = limit(tail(a, Nat.1)).abs
        seq_lte_preserves_limit(abs_seq(tail(a, Nat.1)), const_seq(two * h.abs.pow(Nat.2)))
        limit(abs_seq(tail(a, Nat.1))) <= limit(const_seq(two * h.abs.pow(Nat.2)))
        const_limit(two * h.abs.pow(Nat.2))
        limit(const_seq(two * h.abs.pow(Nat.2))) = two * h.abs.pow(Nat.2)
        limit(abs_seq(tail(a, Nat.1))) <= two * h.abs.pow(Nat.2)
        limit(tail(a, Nat.1)).abs <= two * h.abs.pow(Nat.2)
        convergent_converges_to_limit(tail(a, Nat.1))
        converges_to(tail(a, Nat.1), limit(tail(a, Nat.1)))
        converges_to_unique(tail(a, Nat.1), limit(tail(a, Nat.1)), limit(a))
        limit(tail(a, Nat.1)) = limit(a)
        limit(a).abs <= two * h.abs.pow(Nat.2)
        convergent_converges_to_limit(a)
        converges_to(a, limit(a))
        forall(n: Nat) {
            a(n) = partial(sin_shift_term(h), n) - Real.1
        }
        add_seq(partial(sin_shift_term(h)), neg_seq(const_seq(Real.1))) = a
        limit(a) = limit(add_seq(partial(sin_shift_term(h)), neg_seq(const_seq(Real.1))))
        limit_add_seq(partial(sin_shift_term(h)), neg_seq(const_seq(Real.1)))
        converges_to(add_seq(partial(sin_shift_term(h)), neg_seq(const_seq(Real.1))),
            limit(partial(sin_shift_term(h))) + limit(neg_seq(const_seq(Real.1))))
        convergent_converges_to_limit(add_seq(partial(sin_shift_term(h)), neg_seq(const_seq(Real.1))))
        converges_to(add_seq(partial(sin_shift_term(h)), neg_seq(const_seq(Real.1))),
            limit(add_seq(partial(sin_shift_term(h)), neg_seq(const_seq(Real.1)))))
        converges_to_unique(add_seq(partial(sin_shift_term(h)), neg_seq(const_seq(Real.1))),
            limit(add_seq(partial(sin_shift_term(h)), neg_seq(const_seq(Real.1)))),
            limit(partial(sin_shift_term(h))) + limit(neg_seq(const_seq(Real.1))))
        limit(add_seq(partial(sin_shift_term(h)), neg_seq(const_seq(Real.1)))) =
            limit(partial(sin_shift_term(h))) + limit(neg_seq(const_seq(Real.1)))
        neg_seq_converges_to(const_seq(Real.1))
        converges_to(neg_seq(const_seq(Real.1)), -limit(const_seq(Real.1)))
        converges(neg_seq(const_seq(Real.1)))
        convergent_converges_to_limit(neg_seq(const_seq(Real.1)))
        converges_to(neg_seq(const_seq(Real.1)), limit(neg_seq(const_seq(Real.1))))
        converges_to_unique(neg_seq(const_seq(Real.1)), limit(neg_seq(const_seq(Real.1))),
            -limit(const_seq(Real.1)))
        limit(neg_seq(const_seq(Real.1))) = -limit(const_seq(Real.1))
        const_limit(Real.1)
        limit(const_seq(Real.1)) = Real.1
        limit(neg_seq(const_seq(Real.1))) = -Real.1
        limit(add_seq(partial(sin_shift_term(h)), neg_seq(const_seq(Real.1)))) =
            limit(partial(sin_shift_term(h))) - Real.1
        limit(a) = limit(partial(sin_shift_term(h))) - Real.1
        sin_shift_sum(h) = limit(partial(sin_shift_term(h)))
        limit(a) = sin_shift_sum(h) - Real.1
        limit(a).abs <= two * h.abs.pow(Nat.2)
        (sin_shift_sum(h) - Real.1).abs <= two * h.abs.pow(Nat.2)
    }
}

/// The shifted cosine sum is bounded: |C(h)| <= 2 when |h| <= 1/2.
theorem cos_shift_sum_bound(h: Real) {
    h.abs < Real.1 / two implies cos_shift_sum(h).abs <= two
} by {
    if h.abs < Real.1 / two {
        lt_imp_lte(h.abs, Real.1 / two)
        h.abs <= Real.1 / two
        half_lt_one
        Real.1 / two < Real.1
        lt_of_lte_of_lt(h.abs, Real.1 / two, Real.1)
        h.abs < Real.1
        cos_shift_term_abs_converges_small(h)
        absolutely_converges(cos_shift_term(h))
        absolutely_converges_imp_converges(cos_shift_term(h))
        converges(partial(cos_shift_term(h)))
        forall(n: Nat) {
            partial_abs_le_partial_abs_fn(cos_shift_term(h), n)
            partial(cos_shift_term(h), n).abs <= partial(abs_fn(cos_shift_term(h)), n)
            forall(j: Nat) {
                abs_fn(cos_shift_term(h), j) = cos_shift_term(h, j).abs
                cos_shift_term_abs_le_geom(h, j)
                cos_shift_term(h, j).abs <= h.abs.pow(Nat.2).pow(j)
                abs_fn(cos_shift_term(h), j) <= h.abs.pow(Nat.2).pow(j)
            }
            seq_lte(abs_fn(cos_shift_term(h)), h.abs.pow(Nat.2).pow)
            partial_seq_lte(abs_fn(cos_shift_term(h)), h.abs.pow(Nat.2).pow)
            seq_lte(partial(abs_fn(cos_shift_term(h))), partial(h.abs.pow(Nat.2).pow))
            partial(abs_fn(cos_shift_term(h)), n) <= partial(h.abs.pow(Nat.2).pow, n)
            sq_lte_base(h.abs)
            h.abs.pow(Nat.2) <= h.abs
            lt_of_lte_of_lt(h.abs.pow(Nat.2), h.abs, Real.1)
            h.abs.pow(Nat.2) < Real.1
            abs_of_nonneg(h.abs.pow(Nat.2))
            pow_nonneg(h.abs, Nat.2)
            h.abs.pow(Nat.2) >= Real.0
            h.abs.pow(Nat.2).abs = h.abs.pow(Nat.2)
            h.abs.pow(Nat.2).abs < Real.1
            geom_partial_bound(h.abs.pow(Nat.2), n)
            partial(h.abs.pow(Nat.2).pow, n) <= Real.1 / (Real.1 - h.abs.pow(Nat.2))
            lt_of_lte_of_lt(h.abs.pow(Nat.2), h.abs, Real.1 / two)
            h.abs.pow(Nat.2) < Real.1 / two
            lte_add_right(h.abs.pow(Nat.2), Real.1 / two, -Real.1)
            h.abs.pow(Nat.2) + -Real.1 < Real.1 / two + -Real.1
            h.abs.pow(Nat.2) - Real.1 < Real.1 / two - Real.1
            -(h.abs.pow(Nat.2) - Real.1) > -(Real.1 / two - Real.1)
            Real.1 - h.abs.pow(Nat.2) > Real.1 - Real.1 / two
            one_div_two_eq_half
            Real.1 / two = Real.one_half
            Real.1 - Real.1 / two = Real.1 - Real.one_half
            Real.1 - Real.one_half = Real.one_half
            Real.1 - Real.one_half = Real.1 / two
            Real.1 - Real.1 / two = Real.1 / two
Real.1 - h.abs.pow(Nat.2) > Real.1 / two
            lt_imp_lte(Real.1 / two, Real.1 - h.abs.pow(Nat.2))
            Real.1 / two <= Real.1 - h.abs.pow(Nat.2)
            half_pos
            Real.1 / two > Real.0
            Real.1 - h.abs.pow(Nat.2) > Real.0
            real_recip_antitone_pos(Real.1 / two, Real.1 - h.abs.pow(Nat.2))
            Real.1 / (Real.1 - h.abs.pow(Nat.2)) <= Real.1 / (Real.1 / two)
            one_div_one_half_eq_two
            Real.1 / Real.one_half = two
            Real.1 / two = Real.one_half
            Real.1 / (Real.1 / two) = two
            Real.1 / (Real.1 - h.abs.pow(Nat.2)) <= two
            lte_trans(partial(h.abs.pow(Nat.2).pow, n), Real.1 / (Real.1 - h.abs.pow(Nat.2)), two)
            partial(h.abs.pow(Nat.2).pow, n) <= two
            lte_trans(partial(abs_fn(cos_shift_term(h)), n), partial(h.abs.pow(Nat.2).pow, n), two)
            partial(abs_fn(cos_shift_term(h)), n) <= two
            lte_trans(partial(cos_shift_term(h), n).abs, partial(abs_fn(cos_shift_term(h)), n), two)
            partial(cos_shift_term(h), n).abs <= two
            abs_seq(partial(cos_shift_term(h)))(n) = partial(cos_shift_term(h), n).abs
            abs_seq(partial(cos_shift_term(h)))(n) <= two
            const_seq(two, n) = two
            abs_seq(partial(cos_shift_term(h)))(n) <= const_seq(two, n)
        }
        seq_lte(abs_seq(partial(cos_shift_term(h))), const_seq(two))
        const_converges(two)
        converges(const_seq(two))
        limit_abs_seq(partial(cos_shift_term(h)))
        converges_to(abs_seq(partial(cos_shift_term(h))), limit(partial(cos_shift_term(h))).abs)
        converges(abs_seq(partial(cos_shift_term(h))))
        convergent_converges_to_limit(abs_seq(partial(cos_shift_term(h))))
        converges_to(abs_seq(partial(cos_shift_term(h))), limit(abs_seq(partial(cos_shift_term(h)))))
        converges_to_unique(abs_seq(partial(cos_shift_term(h))), limit(abs_seq(partial(cos_shift_term(h)))),
            limit(partial(cos_shift_term(h))).abs)
        limit(abs_seq(partial(cos_shift_term(h)))) = limit(partial(cos_shift_term(h))).abs
        seq_lte_preserves_limit(abs_seq(partial(cos_shift_term(h))), const_seq(two))
        limit(abs_seq(partial(cos_shift_term(h)))) <= limit(const_seq(two))
        const_limit(two)
        limit(const_seq(two)) = two
        limit(abs_seq(partial(cos_shift_term(h)))) <= two
        limit(partial(cos_shift_term(h))).abs <= two
        cos_shift_sum(h) = limit(partial(cos_shift_term(h)))
        cos_shift_sum(h).abs <= two
    }
}

// Derivatives of sine and cosine.

/// The sine function has derivative x0.cos at every point x0.
theorem sin_has_derivative_at(x0: Real) {
    has_derivative_at(Real.sin, x0, x0.cos)
} by {
    forall(eps: Real) {
        if eps.is_positive {
            exists_small_mul_variant(two * (x0.sin.abs + x0.cos.abs), eps)
            let delta1: Real satisfy {
                delta1.is_positive and (two * (x0.sin.abs + x0.cos.abs)).abs * delta1 < eps
            }
            half_pos
            Real.1 / two > Real.0
            eps_smaller_than_both(Real.1 / two, delta1)
            let delta: Real satisfy {
                delta.is_positive and delta < Real.1 / two and delta < delta1
            }
            forall(x: Real) {
                if x != x0 and x.is_close(x0, delta) {
                    sub_ne_zero_of_ne(x, x0)
                    x - x0 != Real.0
                    x.is_close(x0, delta) = (x - x0).abs < delta
                    (x - x0).abs < delta
                    lt_trans((x - x0).abs, delta, Real.1 / two)
                    (x - x0).abs < Real.1 / two
                    lt_trans((x - x0).abs, delta, delta1)
                    (x - x0).abs < delta1
                    x - x0 = x + -x0
                    x0 + (x - x0) = x0 + (x + -x0)
                    add_comm(x0, x + -x0)
                    x0 + (x + -x0) = (x + -x0) + x0
                    (x + -x0) + x0 = x + (-x0 + x0)
                    -x0 + x0 = Real.0
                    x + (-x0 + x0) = x + Real.0
                    x + Real.0 = x
                    x0 + (x - x0) = x
                    sin_add(x0, x - x0)
                    (x0 + (x - x0)).sin = x0.sin * (x - x0).cos + x0.cos * (x - x0).sin
                    x.sin = x0.sin * (x - x0).cos + x0.cos * (x - x0).sin
                    x.sin - x0.sin = x0.sin * (x - x0).cos + x0.cos * (x - x0).sin - x0.sin
                    x0.sin * (x - x0).cos + x0.cos * (x - x0).sin - x0.sin =
                        x0.sin * ((x - x0).cos - Real.1) + x0.cos * (x - x0).sin
                    x.sin - x0.sin = x0.sin * ((x - x0).cos - Real.1) + x0.cos * (x - x0).sin
                    lt_trans((x - x0).abs, Real.1 / two, Real.1)
                    (x - x0).abs < Real.1
                    cos_sub_one_series_small(x - x0)
                    (x - x0).cos - Real.1 = (x - x0).pow(Nat.2) * cos_shift_sum(x - x0)
                    sin_series_small(x - x0)
                    (x - x0).sin = (x - x0) * sin_shift_sum(x - x0)
                    x0.sin * ((x - x0).cos - Real.1) + x0.cos * (x - x0).sin =
                        x0.sin * ((x - x0).pow(Nat.2) * cos_shift_sum(x - x0)) + x0.cos * ((x - x0) * sin_shift_sum(x - x0))
                    x.sin - x0.sin =
                        x0.sin * ((x - x0).pow(Nat.2) * cos_shift_sum(x - x0)) + x0.cos * ((x - x0) * sin_shift_sum(x - x0))
                    difference_quotient(Real.sin, x0, x) = (x.sin - x0.sin) / (x - x0)
                    difference_quotient(Real.sin, x0, x) =
                        (x0.sin * ((x - x0).pow(Nat.2) * cos_shift_sum(x - x0)) + x0.cos * ((x - x0) * sin_shift_sum(x - x0))) / (x - x0)
                    div_add_distrib(x0.sin * ((x - x0).pow(Nat.2) * cos_shift_sum(x - x0)), x0.cos * ((x - x0) * sin_shift_sum(x - x0)), x - x0)
                    (x0.sin * ((x - x0).pow(Nat.2) * cos_shift_sum(x - x0)) + x0.cos * ((x - x0) * sin_shift_sum(x - x0))) / (x - x0) =
                        (x0.sin * ((x - x0).pow(Nat.2) * cos_shift_sum(x - x0))) / (x - x0) + (x0.cos * ((x - x0) * sin_shift_sum(x - x0))) / (x - x0)
                    (x0.sin * ((x - x0).pow(Nat.2) * cos_shift_sum(x - x0))) / (x - x0) =
                        x0.sin * (((x - x0).pow(Nat.2) * cos_shift_sum(x - x0)) / (x - x0))
                                        pow_suc(x - x0, Nat.1)
                    (x - x0).pow(Nat.1.suc) = (x - x0) * (x - x0).pow(Nat.1)
                    pow_one[Real](x - x0)
                    (x - x0).pow(Nat.1) = x - x0
                    Nat.1.suc = Nat.2
                    (x - x0).pow(Nat.2) = (x - x0) * (x - x0)
                    (x - x0).pow(Nat.2) * cos_shift_sum(x - x0) = ((x - x0) * (x - x0)) * cos_shift_sum(x - x0)
                    ((x - x0) * (x - x0)) * cos_shift_sum(x - x0) = (x - x0) * ((x - x0) * cos_shift_sum(x - x0))
                    ((x - x0).pow(Nat.2) * cos_shift_sum(x - x0)) / (x - x0) = ((x - x0) * ((x - x0) * cos_shift_sum(x - x0))) / (x - x0)
                    ((x - x0) * ((x - x0) * cos_shift_sum(x - x0))) / (x - x0) = (x - x0) * cos_shift_sum(x - x0)
                    (x0.cos * ((x - x0) * sin_shift_sum(x - x0))) / (x - x0) =
                        x0.cos * (((x - x0) * sin_shift_sum(x - x0)) / (x - x0))
                    ((x - x0) * sin_shift_sum(x - x0)) / (x - x0) = sin_shift_sum(x - x0)
                    difference_quotient(Real.sin, x0, x) =
                        x0.sin * ((x - x0) * cos_shift_sum(x - x0)) + x0.cos * sin_shift_sum(x - x0)
                    x0.sin * ((x - x0) * cos_shift_sum(x - x0)) = x0.sin * (x - x0) * cos_shift_sum(x - x0)
                    difference_quotient(Real.sin, x0, x) =
                        x0.sin * (x - x0) * cos_shift_sum(x - x0) + x0.cos * sin_shift_sum(x - x0)
                    difference_quotient(Real.sin, x0, x) - x0.cos =
                        (x0.sin * (x - x0) * cos_shift_sum(x - x0) + x0.cos * sin_shift_sum(x - x0)) - x0.cos
                    x0.cos * sin_shift_sum(x - x0) - x0.cos * Real.1 = x0.cos * (sin_shift_sum(x - x0) - Real.1)
                    x0.cos * Real.1 = x0.cos
                    x0.cos * sin_shift_sum(x - x0) - x0.cos = x0.cos * (sin_shift_sum(x - x0) - Real.1)
                    (x0.sin * (x - x0) * cos_shift_sum(x - x0) + x0.cos * sin_shift_sum(x - x0)) - x0.cos =
                        (x0.sin * (x - x0) * cos_shift_sum(x - x0) + x0.cos * sin_shift_sum(x - x0)) + -x0.cos
                    add_assoc(x0.sin * (x - x0) * cos_shift_sum(x - x0), x0.cos * sin_shift_sum(x - x0), -x0.cos)
                    (x0.sin * (x - x0) * cos_shift_sum(x - x0) + x0.cos * sin_shift_sum(x - x0)) + -x0.cos =
                        x0.sin * (x - x0) * cos_shift_sum(x - x0) + (x0.cos * sin_shift_sum(x - x0) + -x0.cos)
                    x0.sin * (x - x0) * cos_shift_sum(x - x0) + (x0.cos * sin_shift_sum(x - x0) + -x0.cos) =
                        x0.sin * (x - x0) * cos_shift_sum(x - x0) + (x0.cos * sin_shift_sum(x - x0) - x0.cos)
                    x0.sin * (x - x0) * cos_shift_sum(x - x0) + (x0.cos * sin_shift_sum(x - x0) - x0.cos) =
                        x0.sin * (x - x0) * cos_shift_sum(x - x0) + x0.cos * (sin_shift_sum(x - x0) - Real.1)
                    difference_quotient(Real.sin, x0, x) - x0.cos =
                        x0.sin * (x - x0) * cos_shift_sum(x - x0) + x0.cos * (sin_shift_sum(x - x0) - Real.1)
                    triangle_ineq(x0.sin * (x - x0) * cos_shift_sum(x - x0), x0.cos * (sin_shift_sum(x - x0) - Real.1))
                    (difference_quotient(Real.sin, x0, x) - x0.cos).abs <= (x0.sin * (x - x0) * cos_shift_sum(x - x0)).abs + (x0.cos * (sin_shift_sum(x - x0) - Real.1)).abs
                    mul_abs(x0.sin, (x - x0) * cos_shift_sum(x - x0))
                    (x0.sin * ((x - x0) * cos_shift_sum(x - x0))).abs = x0.sin.abs * ((x - x0) * cos_shift_sum(x - x0)).abs
                    x0.sin * (x - x0) * cos_shift_sum(x - x0) = x0.sin * ((x - x0) * cos_shift_sum(x - x0))
                    (x0.sin * (x - x0) * cos_shift_sum(x - x0)).abs = x0.sin.abs * ((x - x0) * cos_shift_sum(x - x0)).abs
                    mul_abs(x - x0, cos_shift_sum(x - x0))
                    ((x - x0) * cos_shift_sum(x - x0)).abs = (x - x0).abs * cos_shift_sum(x - x0).abs
                    (x0.sin * (x - x0) * cos_shift_sum(x - x0)).abs =
                        x0.sin.abs * (x - x0).abs * cos_shift_sum(x - x0).abs
                    mul_abs(x0.cos, sin_shift_sum(x - x0) - Real.1)
                    (x0.cos * (sin_shift_sum(x - x0) - Real.1)).abs =
                        x0.cos.abs * (sin_shift_sum(x - x0) - Real.1).abs
                    (difference_quotient(Real.sin, x0, x) - x0.cos).abs <= x0.sin.abs * (x - x0).abs * cos_shift_sum(x - x0).abs + x0.cos.abs * (sin_shift_sum(x - x0) - Real.1).abs
                    cos_shift_sum_bound(x - x0)
                    cos_shift_sum(x - x0).abs <= two
                    abs_gte_zero(x0.sin)
                    abs_gte_zero(x - x0)
                    x0.sin.abs >= Real.0
                    (x - x0).abs >= Real.0
                    lte_mul_nonneg_right(cos_shift_sum(x - x0).abs, two, (x - x0).abs)
                    cos_shift_sum(x - x0).abs * (x - x0).abs <= two * (x - x0).abs
                    (x - x0).abs * cos_shift_sum(x - x0).abs <= (x - x0).abs * two
                    lte_mul_nonneg_right((x - x0).abs * cos_shift_sum(x - x0).abs, (x - x0).abs * two, x0.sin.abs)
                    x0.sin.abs * ((x - x0).abs * cos_shift_sum(x - x0).abs) <= x0.sin.abs * ((x - x0).abs * two)
                    x0.sin.abs * (x - x0).abs * cos_shift_sum(x - x0).abs <= x0.sin.abs * (x - x0).abs * two
                    x0.sin.abs * (x - x0).abs * two = two * x0.sin.abs * (x - x0).abs
                    x0.sin.abs * (x - x0).abs * cos_shift_sum(x - x0).abs <= two * x0.sin.abs * (x - x0).abs
                    sin_shift_sum_close_one(x - x0)
                    (sin_shift_sum(x - x0) - Real.1).abs <= two * (x - x0).abs.pow(Nat.2)
                    abs_gte_zero(x0.cos)
                    x0.cos.abs >= Real.0
                    lte_mul_nonneg_right((sin_shift_sum(x - x0) - Real.1).abs, two * (x - x0).abs.pow(Nat.2), x0.cos.abs)
                    x0.cos.abs * (sin_shift_sum(x - x0) - Real.1).abs <= x0.cos.abs * (two * (x - x0).abs.pow(Nat.2))
                                        abs_gte_zero(x - x0)
                    (x - x0).abs >= Real.0
                    lt_imp_lte((x - x0).abs, Real.1 / two)
                    (x - x0).abs <= Real.1 / two
                    half_lt_one
                    Real.1 / two < Real.1
                    lt_of_lte_of_lt((x - x0).abs, Real.1 / two, Real.1)
                    (x - x0).abs < Real.1
                    lt_imp_lte((x - x0).abs, Real.1)
                    (x - x0).abs <= Real.1
                    sq_lte_base((x - x0).abs)
                    (x - x0).abs.pow(Nat.2) <= (x - x0).abs
                    two_positive
                    two > Real.0
                    two >= Real.0
                    lte_mul_nonneg_right((x - x0).abs.pow(Nat.2), (x - x0).abs, two)
                    (x - x0).abs.pow(Nat.2) * two <= (x - x0).abs * two
                    two * (x - x0).abs.pow(Nat.2) <= two * (x - x0).abs
                    lte_mul_nonneg_right(two * (x - x0).abs.pow(Nat.2), two * (x - x0).abs, x0.cos.abs)
                    x0.cos.abs * (two * (x - x0).abs.pow(Nat.2)) <= x0.cos.abs * (two * (x - x0).abs)
                    lte_trans(x0.cos.abs * (sin_shift_sum(x - x0) - Real.1).abs,
                        x0.cos.abs * (two * (x - x0).abs.pow(Nat.2)), x0.cos.abs * (two * (x - x0).abs))
                    x0.cos.abs * (sin_shift_sum(x - x0) - Real.1).abs <= x0.cos.abs * (two * (x - x0).abs)
                    x0.cos.abs * (two * (x - x0).abs) = two * x0.cos.abs * (x - x0).abs
                    x0.cos.abs * (sin_shift_sum(x - x0) - Real.1).abs <= two * x0.cos.abs * (x - x0).abs
                    lte_add_right(x0.sin.abs * (x - x0).abs * cos_shift_sum(x - x0).abs,
                        two * x0.sin.abs * (x - x0).abs, x0.cos.abs * (sin_shift_sum(x - x0) - Real.1).abs)
                    x0.sin.abs * (x - x0).abs * cos_shift_sum(x - x0).abs + x0.cos.abs * (sin_shift_sum(x - x0) - Real.1).abs <= two * x0.sin.abs * (x - x0).abs + x0.cos.abs * (sin_shift_sum(x - x0) - Real.1).abs
                    lte_add_right(x0.cos.abs * (sin_shift_sum(x - x0) - Real.1).abs,
                        two * x0.cos.abs * (x - x0).abs, two * x0.sin.abs * (x - x0).abs)
                    two * x0.sin.abs * (x - x0).abs + x0.cos.abs * (sin_shift_sum(x - x0) - Real.1).abs <= two * x0.sin.abs * (x - x0).abs + two * x0.cos.abs * (x - x0).abs
                    lte_trans(x0.sin.abs * (x - x0).abs * cos_shift_sum(x - x0).abs + x0.cos.abs * (sin_shift_sum(x - x0) - Real.1).abs,
                        two * x0.sin.abs * (x - x0).abs + x0.cos.abs * (sin_shift_sum(x - x0) - Real.1).abs,
                        two * x0.sin.abs * (x - x0).abs + two * x0.cos.abs * (x - x0).abs)
                    x0.sin.abs * (x - x0).abs * cos_shift_sum(x - x0).abs + x0.cos.abs * (sin_shift_sum(x - x0) - Real.1).abs <= two * x0.sin.abs * (x - x0).abs + two * x0.cos.abs * (x - x0).abs
                    two * x0.sin.abs * (x - x0).abs + two * x0.cos.abs * (x - x0).abs =
                        (two * x0.sin.abs + two * x0.cos.abs) * (x - x0).abs
                    two * x0.sin.abs + two * x0.cos.abs = two * (x0.sin.abs + x0.cos.abs)
                    (two * x0.sin.abs + two * x0.cos.abs) * (x - x0).abs = (two * (x0.sin.abs + x0.cos.abs)) * (x - x0).abs
                    two * x0.sin.abs * (x - x0).abs + two * x0.cos.abs * (x - x0).abs = (two * (x0.sin.abs + x0.cos.abs)) * (x - x0).abs
                    two * x0.sin.abs * (x - x0).abs + two * x0.cos.abs * (x - x0).abs <= (two * (x0.sin.abs + x0.cos.abs)) * (x - x0).abs
                    lte_trans(x0.sin.abs * (x - x0).abs * cos_shift_sum(x - x0).abs + x0.cos.abs * (sin_shift_sum(x - x0) - Real.1).abs,
                        two * x0.sin.abs * (x - x0).abs + two * x0.cos.abs * (x - x0).abs,
                        (two * (x0.sin.abs + x0.cos.abs)) * (x - x0).abs)
                    x0.sin.abs * (x - x0).abs * cos_shift_sum(x - x0).abs + x0.cos.abs * (sin_shift_sum(x - x0) - Real.1).abs <= (two * (x0.sin.abs + x0.cos.abs)) * (x - x0).abs
                    lte_trans((difference_quotient(Real.sin, x0, x) - x0.cos).abs,
                        x0.sin.abs * (x - x0).abs * cos_shift_sum(x - x0).abs + x0.cos.abs * (sin_shift_sum(x - x0) - Real.1).abs,
                        (two * (x0.sin.abs + x0.cos.abs)) * (x - x0).abs)
                    (difference_quotient(Real.sin, x0, x) - x0.cos).abs <= (two * (x0.sin.abs + x0.cos.abs)) * (x - x0).abs
                    lte_abs(two * (x0.sin.abs + x0.cos.abs))
                    two * (x0.sin.abs + x0.cos.abs) <= (two * (x0.sin.abs + x0.cos.abs)).abs
                    lte_mul_nonneg_right(two * (x0.sin.abs + x0.cos.abs),
                        (two * (x0.sin.abs + x0.cos.abs)).abs, (x - x0).abs)
                    (two * (x0.sin.abs + x0.cos.abs)) * (x - x0).abs <= (two * (x0.sin.abs + x0.cos.abs)).abs * (x - x0).abs
                    lte_trans((difference_quotient(Real.sin, x0, x) - x0.cos).abs,
                        (two * (x0.sin.abs + x0.cos.abs)) * (x - x0).abs,
                        (two * (x0.sin.abs + x0.cos.abs)).abs * (x - x0).abs)
                    (difference_quotient(Real.sin, x0, x) - x0.cos).abs <= (two * (x0.sin.abs + x0.cos.abs)).abs * (x - x0).abs
                                        lt_imp_lte((x - x0).abs, delta1)
                    (x - x0).abs <= delta1
                    lte_mul_nonneg_right((x - x0).abs, delta1, (two * (x0.sin.abs + x0.cos.abs)).abs)
                    (x - x0).abs * (two * (x0.sin.abs + x0.cos.abs)).abs <= delta1 * (two * (x0.sin.abs + x0.cos.abs)).abs
                    (two * (x0.sin.abs + x0.cos.abs)).abs * (x - x0).abs <= (two * (x0.sin.abs + x0.cos.abs)).abs * delta1
                    lte_trans((difference_quotient(Real.sin, x0, x) - x0.cos).abs,
                        (two * (x0.sin.abs + x0.cos.abs)).abs * (x - x0).abs,
                        (two * (x0.sin.abs + x0.cos.abs)).abs * delta1)
                    (difference_quotient(Real.sin, x0, x) - x0.cos).abs <= (two * (x0.sin.abs + x0.cos.abs)).abs * delta1
                    lte_lt_trans((difference_quotient(Real.sin, x0, x) - x0.cos).abs,
                        (two * (x0.sin.abs + x0.cos.abs)).abs * delta1, eps)
                    (difference_quotient(Real.sin, x0, x) - x0.cos).abs < eps
                    difference_quotient(Real.sin, x0, x).is_close(x0.cos, eps)
                }
            }
            exists(delta2: Real) {
                delta2.is_positive and forall(x: Real) {
                    x != x0 and x.is_close(x0, delta2)
                    implies difference_quotient(Real.sin, x0, x).is_close(x0.cos, eps)
                }
            }
        }
    }
    has_derivative_at(Real.sin, x0, x0.cos) = forall(eps: Real) {
        eps.is_positive implies exists(delta: Real) {
            delta.is_positive and forall(x: Real) {
                x != x0 and x.is_close(x0, delta)
                implies difference_quotient(Real.sin, x0, x).is_close(x0.cos, eps)
            }
        }
    }
    if not has_derivative_at(Real.sin, x0, x0.cos) {
        not forall(eps: Real) {
            eps.is_positive implies exists(delta: Real) {
                delta.is_positive and forall(x: Real) {
                    x != x0 and x.is_close(x0, delta)
                    implies difference_quotient(Real.sin, x0, x).is_close(x0.cos, eps)
                }
            }
        }
        let bad_eps: Real satisfy {
            bad_eps.is_positive and forall(delta: Real) {
                not (delta.is_positive and forall(x: Real) {
                    x != x0 and x.is_close(x0, delta)
                    implies difference_quotient(Real.sin, x0, x).is_close(x0.cos, bad_eps)
                })
            }
        }
        exists(delta: Real) {
            delta.is_positive and forall(x: Real) {
                x != x0 and x.is_close(x0, delta)
                implies difference_quotient(Real.sin, x0, x).is_close(x0.cos, bad_eps)
            }
        }
        false
    }
}

/// The sine function is everywhere differentiable with derivative cosine.
theorem sin_is_derivative_fn {
    is_derivative_fn(Real.sin, Real.cos)
} by {
    forall(x: Real) {
        sin_has_derivative_at(x)
        has_derivative_at(Real.sin, x, x.cos)
    }
    is_derivative_fn(Real.sin, Real.cos) = forall(y: Real) {
        has_derivative_at(Real.sin, y, y.cos)
    }
    is_derivative_fn(Real.sin, Real.cos)
}

/// The cosine function has derivative -x0.sin at every point x0.
theorem cos_has_derivative_at(x0: Real) {
    has_derivative_at(Real.cos, x0, -x0.sin)
} by {
    forall(eps: Real) {
        if eps.is_positive {
            exists_small_mul_variant(two * (x0.sin.abs + x0.cos.abs), eps)
            let delta1: Real satisfy {
                delta1.is_positive and (two * (x0.sin.abs + x0.cos.abs)).abs * delta1 < eps
            }
            half_pos
            Real.1 / two > Real.0
            eps_smaller_than_both(Real.1 / two, delta1)
            let delta: Real satisfy {
                delta.is_positive and delta < Real.1 / two and delta < delta1
            }
            forall(x: Real) {
                if x != x0 and x.is_close(x0, delta) {
                    sub_ne_zero_of_ne(x, x0)
                    x - x0 != Real.0
                    x.is_close(x0, delta) = (x - x0).abs < delta
                    (x - x0).abs < delta
                    lt_trans((x - x0).abs, delta, Real.1 / two)
                    (x - x0).abs < Real.1 / two
                    lt_trans((x - x0).abs, delta, delta1)
                    (x - x0).abs < delta1
                    x - x0 = x + -x0
                    x0 + (x - x0) = x0 + (x + -x0)
                    add_comm(x0, x + -x0)
                    x0 + (x + -x0) = (x + -x0) + x0
                    (x + -x0) + x0 = x + (-x0 + x0)
                    -x0 + x0 = Real.0
                    x + (-x0 + x0) = x + Real.0
                    x + Real.0 = x
                    x0 + (x - x0) = x
                    cos_add(x0, x - x0)
                    (x0 + (x - x0)).cos = x0.cos * (x - x0).cos - x0.sin * (x - x0).sin
                    x.cos = x0.cos * (x - x0).cos - x0.sin * (x - x0).sin
                    x.cos - x0.cos = x0.cos * (x - x0).cos - x0.sin * (x - x0).sin - x0.cos
                    x0.cos * (x - x0).cos - x0.sin * (x - x0).sin - x0.cos =
                        x0.cos * ((x - x0).cos - Real.1) - x0.sin * (x - x0).sin
                    x.cos - x0.cos = x0.cos * ((x - x0).cos - Real.1) - x0.sin * (x - x0).sin
                    lt_trans((x - x0).abs, Real.1 / two, Real.1)
                    (x - x0).abs < Real.1
                    cos_sub_one_series_small(x - x0)
                    (x - x0).cos - Real.1 = (x - x0).pow(Nat.2) * cos_shift_sum(x - x0)
                    sin_series_small(x - x0)
                    (x - x0).sin = (x - x0) * sin_shift_sum(x - x0)
                    x0.cos * ((x - x0).cos - Real.1) - x0.sin * (x - x0).sin =
                        x0.cos * ((x - x0).pow(Nat.2) * cos_shift_sum(x - x0)) - x0.sin * ((x - x0) * sin_shift_sum(x - x0))
                    x.cos - x0.cos =
                        x0.cos * ((x - x0).pow(Nat.2) * cos_shift_sum(x - x0)) - x0.sin * ((x - x0) * sin_shift_sum(x - x0))
                    difference_quotient(Real.cos, x0, x) = (x.cos - x0.cos) / (x - x0)
                    difference_quotient(Real.cos, x0, x) =
                        (x0.cos * ((x - x0).pow(Nat.2) * cos_shift_sum(x - x0)) - x0.sin * ((x - x0) * sin_shift_sum(x - x0))) / (x - x0)
                    div_add_distrib(x0.cos * ((x - x0).pow(Nat.2) * cos_shift_sum(x - x0)), -(x0.sin * ((x - x0) * sin_shift_sum(x - x0))), x - x0)
                    (x0.cos * ((x - x0).pow(Nat.2) * cos_shift_sum(x - x0)) + -(x0.sin * ((x - x0) * sin_shift_sum(x - x0)))) / (x - x0) =
                        (x0.cos * ((x - x0).pow(Nat.2) * cos_shift_sum(x - x0))) / (x - x0) + (-(x0.sin * ((x - x0) * sin_shift_sum(x - x0)))) / (x - x0)
                    x0.cos * ((x - x0).pow(Nat.2) * cos_shift_sum(x - x0)) - x0.sin * ((x - x0) * sin_shift_sum(x - x0)) =
                        x0.cos * ((x - x0).pow(Nat.2) * cos_shift_sum(x - x0)) + -(x0.sin * ((x - x0) * sin_shift_sum(x - x0)))
                    (x0.cos * ((x - x0).pow(Nat.2) * cos_shift_sum(x - x0)) - x0.sin * ((x - x0) * sin_shift_sum(x - x0))) / (x - x0) =
                        (x0.cos * ((x - x0).pow(Nat.2) * cos_shift_sum(x - x0))) / (x - x0) + (-(x0.sin * ((x - x0) * sin_shift_sum(x - x0)))) / (x - x0)
                    pow_suc(x - x0, Nat.1)
                    (x - x0).pow(Nat.1.suc) = (x - x0) * (x - x0).pow(Nat.1)
                    pow_one[Real](x - x0)
                    (x - x0).pow(Nat.1) = x - x0
                    Nat.1.suc = Nat.2
                    (x - x0).pow(Nat.2) = (x - x0) * (x - x0)
                    (x - x0).pow(Nat.2) * cos_shift_sum(x - x0) = ((x - x0) * (x - x0)) * cos_shift_sum(x - x0)
                    ((x - x0) * (x - x0)) * cos_shift_sum(x - x0) = (x - x0) * ((x - x0) * cos_shift_sum(x - x0))
                    (x0.cos * ((x - x0).pow(Nat.2) * cos_shift_sum(x - x0))) / (x - x0) =
                        x0.cos * (((x - x0).pow(Nat.2) * cos_shift_sum(x - x0)) / (x - x0))
                    ((x - x0).pow(Nat.2) * cos_shift_sum(x - x0)) / (x - x0) = ((x - x0) * ((x - x0) * cos_shift_sum(x - x0))) / (x - x0)
                    ((x - x0) * ((x - x0) * cos_shift_sum(x - x0))) / (x - x0) = (x - x0) * cos_shift_sum(x - x0)
                    (x0.cos * ((x - x0).pow(Nat.2) * cos_shift_sum(x - x0))) / (x - x0) =
                        x0.cos * ((x - x0) * cos_shift_sum(x - x0))
                    (x0.sin * ((x - x0) * sin_shift_sum(x - x0))) / (x - x0) =
                        x0.sin * (((x - x0) * sin_shift_sum(x - x0)) / (x - x0))
                    ((x - x0) * sin_shift_sum(x - x0)) / (x - x0) = sin_shift_sum(x - x0)
                    (-(x0.sin * ((x - x0) * sin_shift_sum(x - x0)))) / (x - x0) =
                        -(x0.sin * ((x - x0) * sin_shift_sum(x - x0))) / (x - x0)
                    (-(x0.sin * ((x - x0) * sin_shift_sum(x - x0)))) / (x - x0) =
                        -(x0.sin * (((x - x0) * sin_shift_sum(x - x0)) / (x - x0)))
                    (-(x0.sin * ((x - x0) * sin_shift_sum(x - x0)))) / (x - x0) = -(x0.sin * sin_shift_sum(x - x0))
                    difference_quotient(Real.cos, x0, x) =
                        x0.cos * ((x - x0) * cos_shift_sum(x - x0)) + -(x0.sin * sin_shift_sum(x - x0))
                    x0.cos * ((x - x0) * cos_shift_sum(x - x0)) + -(x0.sin * sin_shift_sum(x - x0)) =
                        x0.cos * ((x - x0) * cos_shift_sum(x - x0)) - x0.sin * sin_shift_sum(x - x0)
                    difference_quotient(Real.cos, x0, x) =
                        x0.cos * ((x - x0) * cos_shift_sum(x - x0)) - x0.sin * sin_shift_sum(x - x0)
                    x0.cos * ((x - x0) * cos_shift_sum(x - x0)) = x0.cos * (x - x0) * cos_shift_sum(x - x0)
                    difference_quotient(Real.cos, x0, x) =
                        x0.cos * (x - x0) * cos_shift_sum(x - x0) - x0.sin * sin_shift_sum(x - x0)
                    difference_quotient(Real.cos, x0, x) + x0.sin =
                        (x0.cos * (x - x0) * cos_shift_sum(x - x0) - x0.sin * sin_shift_sum(x - x0)) + x0.sin
                    (x0.cos * (x - x0) * cos_shift_sum(x - x0) - x0.sin * sin_shift_sum(x - x0)) + x0.sin =
                        x0.cos * (x - x0) * cos_shift_sum(x - x0) + (-(x0.sin * sin_shift_sum(x - x0)) + x0.sin)
                    -(x0.sin * sin_shift_sum(x - x0)) + x0.sin = x0.sin - x0.sin * sin_shift_sum(x - x0)
                    x0.sin - x0.sin * sin_shift_sum(x - x0) = x0.sin * (Real.1 - sin_shift_sum(x - x0))
                    x0.sin * (Real.1 - sin_shift_sum(x - x0)) = -(x0.sin * (sin_shift_sum(x - x0) - Real.1))
                    x0.sin - x0.sin * sin_shift_sum(x - x0) = -(x0.sin * (sin_shift_sum(x - x0) - Real.1))
                    -(x0.sin * sin_shift_sum(x - x0)) + x0.sin = -(x0.sin * (sin_shift_sum(x - x0) - Real.1))
                    x0.cos * (x - x0) * cos_shift_sum(x - x0) + (-(x0.sin * sin_shift_sum(x - x0)) + x0.sin) =
                        x0.cos * (x - x0) * cos_shift_sum(x - x0) + -(x0.sin * (sin_shift_sum(x - x0) - Real.1))
                    x0.cos * (x - x0) * cos_shift_sum(x - x0) + -(x0.sin * (sin_shift_sum(x - x0) - Real.1)) =
                        x0.cos * (x - x0) * cos_shift_sum(x - x0) - x0.sin * (sin_shift_sum(x - x0) - Real.1)
                    difference_quotient(Real.cos, x0, x) + x0.sin =
                        x0.cos * (x - x0) * cos_shift_sum(x - x0) - x0.sin * (sin_shift_sum(x - x0) - Real.1)
                    x0.cos * (x - x0) * cos_shift_sum(x - x0) - x0.sin * (sin_shift_sum(x - x0) - Real.1) =
                        x0.cos * (x - x0) * cos_shift_sum(x - x0) + -(x0.sin * (sin_shift_sum(x - x0) - Real.1))
                    difference_quotient(Real.cos, x0, x) + x0.sin =
                        x0.cos * (x - x0) * cos_shift_sum(x - x0) + -(x0.sin * (sin_shift_sum(x - x0) - Real.1))
                    triangle_ineq(x0.cos * (x - x0) * cos_shift_sum(x - x0), -(x0.sin * (sin_shift_sum(x - x0) - Real.1)))
                    (x0.cos * (x - x0) * cos_shift_sum(x - x0) + -(x0.sin * (sin_shift_sum(x - x0) - Real.1))).abs <= (x0.cos * (x - x0) * cos_shift_sum(x - x0)).abs + (-(x0.sin * (sin_shift_sum(x - x0) - Real.1))).abs
                    (difference_quotient(Real.cos, x0, x) + x0.sin).abs <= (x0.cos * (x - x0) * cos_shift_sum(x - x0)).abs + (-(x0.sin * (sin_shift_sum(x - x0) - Real.1))).abs
                    abs_neg(x0.sin * (sin_shift_sum(x - x0) - Real.1))
                    (-(x0.sin * (sin_shift_sum(x - x0) - Real.1))).abs = (x0.sin * (sin_shift_sum(x - x0) - Real.1)).abs
                    mul_abs(x0.sin, sin_shift_sum(x - x0) - Real.1)
                    (x0.sin * (sin_shift_sum(x - x0) - Real.1)).abs = x0.sin.abs * (sin_shift_sum(x - x0) - Real.1).abs
                    mul_abs(x0.cos, (x - x0) * cos_shift_sum(x - x0))
                    (x0.cos * ((x - x0) * cos_shift_sum(x - x0))).abs = x0.cos.abs * ((x - x0) * cos_shift_sum(x - x0)).abs
                    x0.cos * (x - x0) * cos_shift_sum(x - x0) = x0.cos * ((x - x0) * cos_shift_sum(x - x0))
                    (x0.cos * (x - x0) * cos_shift_sum(x - x0)).abs = x0.cos.abs * ((x - x0) * cos_shift_sum(x - x0)).abs
                    mul_abs(x - x0, cos_shift_sum(x - x0))
                    ((x - x0) * cos_shift_sum(x - x0)).abs = (x - x0).abs * cos_shift_sum(x - x0).abs
                    (x0.cos * (x - x0) * cos_shift_sum(x - x0)).abs =
                        x0.cos.abs * (x - x0).abs * cos_shift_sum(x - x0).abs
                    (difference_quotient(Real.cos, x0, x) + x0.sin).abs <= x0.cos.abs * (x - x0).abs * cos_shift_sum(x - x0).abs + x0.sin.abs * (sin_shift_sum(x - x0) - Real.1).abs
                    cos_shift_sum_bound(x - x0)
                    cos_shift_sum(x - x0).abs <= two
                    abs_gte_zero(x0.cos)
                    abs_gte_zero(x - x0)
                    x0.cos.abs >= Real.0
                    (x - x0).abs >= Real.0
                    lte_mul_nonneg_right(cos_shift_sum(x - x0).abs, two, (x - x0).abs)
                    cos_shift_sum(x - x0).abs * (x - x0).abs <= two * (x - x0).abs
                    (x - x0).abs * cos_shift_sum(x - x0).abs <= (x - x0).abs * two
                    lte_mul_nonneg_right((x - x0).abs * cos_shift_sum(x - x0).abs, (x - x0).abs * two, x0.cos.abs)
                    x0.cos.abs * ((x - x0).abs * cos_shift_sum(x - x0).abs) <= x0.cos.abs * ((x - x0).abs * two)
                    x0.cos.abs * (x - x0).abs * cos_shift_sum(x - x0).abs <= x0.cos.abs * (x - x0).abs * two
                    x0.cos.abs * (x - x0).abs * two = two * x0.cos.abs * (x - x0).abs
                    x0.cos.abs * (x - x0).abs * cos_shift_sum(x - x0).abs <= two * x0.cos.abs * (x - x0).abs
                    sin_shift_sum_close_one(x - x0)
                    (sin_shift_sum(x - x0) - Real.1).abs <= two * (x - x0).abs.pow(Nat.2)
                    abs_gte_zero(x0.sin)
                    x0.sin.abs >= Real.0
                    lte_mul_nonneg_right((sin_shift_sum(x - x0) - Real.1).abs, two * (x - x0).abs.pow(Nat.2), x0.sin.abs)
                    x0.sin.abs * (sin_shift_sum(x - x0) - Real.1).abs <= x0.sin.abs * (two * (x - x0).abs.pow(Nat.2))
                    abs_gte_zero(x - x0)
                    (x - x0).abs >= Real.0
                    lt_imp_lte((x - x0).abs, Real.1 / two)
                    (x - x0).abs <= Real.1 / two
                    half_lt_one
                    Real.1 / two < Real.1
                    lt_of_lte_of_lt((x - x0).abs, Real.1 / two, Real.1)
                    (x - x0).abs < Real.1
                    lt_imp_lte((x - x0).abs, Real.1)
                    (x - x0).abs <= Real.1
                    sq_lte_base((x - x0).abs)
                    (x - x0).abs.pow(Nat.2) <= (x - x0).abs
                    two_positive
                    two > Real.0
                    two >= Real.0
                    lte_mul_nonneg_right((x - x0).abs.pow(Nat.2), (x - x0).abs, two)
                    (x - x0).abs.pow(Nat.2) * two <= (x - x0).abs * two
                    two * (x - x0).abs.pow(Nat.2) <= two * (x - x0).abs
                    lte_mul_nonneg_right(two * (x - x0).abs.pow(Nat.2), two * (x - x0).abs, x0.sin.abs)
                    x0.sin.abs * (two * (x - x0).abs.pow(Nat.2)) <= x0.sin.abs * (two * (x - x0).abs)
                    lte_trans(x0.sin.abs * (sin_shift_sum(x - x0) - Real.1).abs,
                        x0.sin.abs * (two * (x - x0).abs.pow(Nat.2)), x0.sin.abs * (two * (x - x0).abs))
                    x0.sin.abs * (sin_shift_sum(x - x0) - Real.1).abs <= x0.sin.abs * (two * (x - x0).abs)
                    x0.sin.abs * (two * (x - x0).abs) = two * x0.sin.abs * (x - x0).abs
                    x0.sin.abs * (sin_shift_sum(x - x0) - Real.1).abs <= two * x0.sin.abs * (x - x0).abs
                    lte_add_right(x0.cos.abs * (x - x0).abs * cos_shift_sum(x - x0).abs,
                        two * x0.cos.abs * (x - x0).abs, x0.sin.abs * (sin_shift_sum(x - x0) - Real.1).abs)
                    x0.cos.abs * (x - x0).abs * cos_shift_sum(x - x0).abs + x0.sin.abs * (sin_shift_sum(x - x0) - Real.1).abs <= two * x0.cos.abs * (x - x0).abs + x0.sin.abs * (sin_shift_sum(x - x0) - Real.1).abs
                    lte_add_right(x0.sin.abs * (sin_shift_sum(x - x0) - Real.1).abs,
                        two * x0.sin.abs * (x - x0).abs, two * x0.cos.abs * (x - x0).abs)
                    two * x0.cos.abs * (x - x0).abs + x0.sin.abs * (sin_shift_sum(x - x0) - Real.1).abs <= two * x0.cos.abs * (x - x0).abs + two * x0.sin.abs * (x - x0).abs
                    lte_trans(x0.cos.abs * (x - x0).abs * cos_shift_sum(x - x0).abs + x0.sin.abs * (sin_shift_sum(x - x0) - Real.1).abs,
                        two * x0.cos.abs * (x - x0).abs + x0.sin.abs * (sin_shift_sum(x - x0) - Real.1).abs,
                        two * x0.cos.abs * (x - x0).abs + two * x0.sin.abs * (x - x0).abs)
                    x0.cos.abs * (x - x0).abs * cos_shift_sum(x - x0).abs + x0.sin.abs * (sin_shift_sum(x - x0) - Real.1).abs <= two * x0.cos.abs * (x - x0).abs + two * x0.sin.abs * (x - x0).abs
                    two * x0.cos.abs * (x - x0).abs + two * x0.sin.abs * (x - x0).abs =
                        (two * x0.cos.abs + two * x0.sin.abs) * (x - x0).abs
                    two * x0.cos.abs + two * x0.sin.abs = two * (x0.cos.abs + x0.sin.abs)
                    two * x0.cos.abs + two * x0.sin.abs = two * (x0.sin.abs + x0.cos.abs)
                    (two * x0.cos.abs + two * x0.sin.abs) * (x - x0).abs = (two * (x0.sin.abs + x0.cos.abs)) * (x - x0).abs
                    two * x0.cos.abs * (x - x0).abs + two * x0.sin.abs * (x - x0).abs = (two * (x0.sin.abs + x0.cos.abs)) * (x - x0).abs
                    two * x0.cos.abs * (x - x0).abs + two * x0.sin.abs * (x - x0).abs <= (two * (x0.sin.abs + x0.cos.abs)) * (x - x0).abs
                    lte_trans(x0.cos.abs * (x - x0).abs * cos_shift_sum(x - x0).abs + x0.sin.abs * (sin_shift_sum(x - x0) - Real.1).abs,
                        two * x0.cos.abs * (x - x0).abs + two * x0.sin.abs * (x - x0).abs,
                        (two * (x0.sin.abs + x0.cos.abs)) * (x - x0).abs)
                    x0.cos.abs * (x - x0).abs * cos_shift_sum(x - x0).abs + x0.sin.abs * (sin_shift_sum(x - x0) - Real.1).abs <= (two * (x0.sin.abs + x0.cos.abs)) * (x - x0).abs
                    lte_trans((difference_quotient(Real.cos, x0, x) + x0.sin).abs,
                        x0.cos.abs * (x - x0).abs * cos_shift_sum(x - x0).abs + x0.sin.abs * (sin_shift_sum(x - x0) - Real.1).abs,
                        (two * (x0.sin.abs + x0.cos.abs)) * (x - x0).abs)
                    (difference_quotient(Real.cos, x0, x) + x0.sin).abs <= (two * (x0.sin.abs + x0.cos.abs)) * (x - x0).abs
                    lte_abs(two * (x0.sin.abs + x0.cos.abs))
                    two * (x0.sin.abs + x0.cos.abs) <= (two * (x0.sin.abs + x0.cos.abs)).abs
                    lte_mul_nonneg_right(two * (x0.sin.abs + x0.cos.abs),
                        (two * (x0.sin.abs + x0.cos.abs)).abs, (x - x0).abs)
                    (two * (x0.sin.abs + x0.cos.abs)) * (x - x0).abs <= (two * (x0.sin.abs + x0.cos.abs)).abs * (x - x0).abs
                    lte_trans((difference_quotient(Real.cos, x0, x) + x0.sin).abs,
                        (two * (x0.sin.abs + x0.cos.abs)) * (x - x0).abs,
                        (two * (x0.sin.abs + x0.cos.abs)).abs * (x - x0).abs)
                    (difference_quotient(Real.cos, x0, x) + x0.sin).abs <= (two * (x0.sin.abs + x0.cos.abs)).abs * (x - x0).abs
                    lt_imp_lte((x - x0).abs, delta1)
                    (x - x0).abs <= delta1
                    lte_mul_nonneg_right((x - x0).abs, delta1, (two * (x0.sin.abs + x0.cos.abs)).abs)
                    (x - x0).abs * (two * (x0.sin.abs + x0.cos.abs)).abs <= delta1 * (two * (x0.sin.abs + x0.cos.abs)).abs
                    (two * (x0.sin.abs + x0.cos.abs)).abs * (x - x0).abs <= (two * (x0.sin.abs + x0.cos.abs)).abs * delta1
                    lte_trans((difference_quotient(Real.cos, x0, x) + x0.sin).abs,
                        (two * (x0.sin.abs + x0.cos.abs)).abs * (x - x0).abs,
                        (two * (x0.sin.abs + x0.cos.abs)).abs * delta1)
                    (difference_quotient(Real.cos, x0, x) + x0.sin).abs <= (two * (x0.sin.abs + x0.cos.abs)).abs * delta1
                    lte_lt_trans((difference_quotient(Real.cos, x0, x) + x0.sin).abs,
                        (two * (x0.sin.abs + x0.cos.abs)).abs * delta1, eps)
                    (difference_quotient(Real.cos, x0, x) + x0.sin).abs < eps
                    (difference_quotient(Real.cos, x0, x) - (-x0.sin)).abs < eps
                    difference_quotient(Real.cos, x0, x).is_close(-x0.sin, eps)
                }
            }
            exists(delta2: Real) {
                delta2.is_positive and forall(x: Real) {
                    x != x0 and x.is_close(x0, delta2)
                    implies difference_quotient(Real.cos, x0, x).is_close(-x0.sin, eps)
                }
            }
        }
    }
    has_derivative_at(Real.cos, x0, -x0.sin) = forall(eps: Real) {
        eps.is_positive implies exists(delta: Real) {
            delta.is_positive and forall(x: Real) {
                x != x0 and x.is_close(x0, delta)
                implies difference_quotient(Real.cos, x0, x).is_close(-x0.sin, eps)
            }
        }
    }
    if not has_derivative_at(Real.cos, x0, -x0.sin) {
        not forall(eps: Real) {
            eps.is_positive implies exists(delta: Real) {
                delta.is_positive and forall(x: Real) {
                    x != x0 and x.is_close(x0, delta)
                    implies difference_quotient(Real.cos, x0, x).is_close(-x0.sin, eps)
                }
            }
        }
        let bad_eps: Real satisfy {
            bad_eps.is_positive and forall(delta: Real) {
                not (delta.is_positive and forall(x: Real) {
                    x != x0 and x.is_close(x0, delta)
                    implies difference_quotient(Real.cos, x0, x).is_close(-x0.sin, bad_eps)
                })
            }
        }
        exists(delta: Real) {
            delta.is_positive and forall(x: Real) {
                x != x0 and x.is_close(x0, delta)
                implies difference_quotient(Real.cos, x0, x).is_close(-x0.sin, bad_eps)
            }
        }
        false
    }
}

/// The cosine function is everywhere differentiable with derivative minus sine.
theorem cos_is_derivative_fn {
    is_derivative_fn(Real.cos, pointwise_neg(Real.sin))
} by {
    forall(x: Real) {
        cos_has_derivative_at(x)
        has_derivative_at(Real.cos, x, -x.sin)
        pointwise_neg(Real.sin, x) = -x.sin
        has_derivative_at(Real.cos, x, pointwise_neg(Real.sin, x))
    }
    is_derivative_fn(Real.cos, pointwise_neg(Real.sin)) = forall(y: Real) {
        has_derivative_at(Real.cos, y, pointwise_neg(Real.sin, y))
    }
    is_derivative_fn(Real.cos, pointwise_neg(Real.sin))
}

/// The cosine function is everywhere differentiable with derivative negative
/// sine (uniquely named surface for the interface; see cos_is_derivative_fn).
theorem cos_derivative_is_neg_sin {
    is_derivative_fn(Real.cos, pointwise_neg(Real.sin))
} by {
    cos_is_derivative_fn
    is_derivative_fn(Real.cos, pointwise_neg(Real.sin))
}
