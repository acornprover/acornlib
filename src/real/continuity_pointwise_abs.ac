from real.continuity_composition import continuous_at_delta, continuous_at_intro,
    continuous_intro
from real.continuity_base import Real, continuous, continuous_at, continuous_condition
from real.limits import abs_close_of_close

/// The pointwise absolute value of a real function.
define pointwise_abs(f: Real -> Real, x: Real) -> Real {
    f(x).abs
}

/// Pointwise absolute value preserves continuity at a point.
theorem continuous_at_pointwise_abs(f: Real -> Real, x: Real) {
    continuous_at(f, x) implies continuous_at(pointwise_abs(f), x)
} by {
    forall(eps: Real) {
        if eps.is_positive {
            continuous_at_delta(f, x, eps)
            let delta: Real satisfy {
                delta.is_positive and continuous_condition(f, x, delta, eps)
            }
            continuous_condition(f, x, delta, eps) = forall(y: Real) {
                y.is_close(x, delta) implies f(y).is_close(f(x), eps)
            }
            forall(y: Real) {
                if y.is_close(x, delta) {
                    f(y).is_close(f(x), eps)
                    abs_close_of_close(f(y), f(x), eps)
                    f(y).abs.is_close(f(x).abs, eps)
                    pointwise_abs(f, y) = f(y).abs
                    pointwise_abs(f, x) = f(x).abs
                    pointwise_abs(f, y).is_close(pointwise_abs(f, x), eps)
                }
            }
            continuous_condition(pointwise_abs(f), x, delta, eps)
            delta.is_positive and continuous_condition(pointwise_abs(f), x, delta, eps)
            exists(d: Real) {
                d.is_positive and continuous_condition(pointwise_abs(f), x, d, eps)
            }
        }
    }
    continuous_at_intro(pointwise_abs(f), x)
    continuous_at(pointwise_abs(f), x)
}

/// Pointwise absolute value preserves continuous real functions.
theorem continuous_pointwise_abs(f: Real -> Real) {
    continuous(f) implies continuous(pointwise_abs(f))
} by {
    continuous(f) = forall(x: Real) {
        continuous_at(f, x)
    }
    forall(x: Real) {
        continuous_at(f, x)
        continuous_at_pointwise_abs(f, x)
        continuous_at(pointwise_abs(f), x)
    }
    continuous_intro(pointwise_abs(f))
    continuous(pointwise_abs(f))
}
