from data.basic.function_algebra import pointwise_mul
from data.basic.functions import identity_fn
from real.continuity_pointwise_mul import continuous_at_pointwise_mul,
    continuous_pointwise_mul
from real.continuity_composition import continuous_imp_continuous_at
from real.continuity_sequences import identity_function_is_continuous
from real.continuity_cube import continuous_at_cube_real,
    continuous_cube_real, cube_real
from real.continuity_base import Real, continuous, continuous_at

/// The pointwise fourth power of a real number.
define quartic_real(x: Real) -> Real {
    x * x * x * x
}

/// The quartic function agrees with the pointwise product of the cube with the identity.
theorem quartic_real_eq_pointwise_mul_cube_identity {
    quartic_real = pointwise_mul[Real, Real](cube_real, identity_fn[Real])
} by {
    forall(x: Real) {
        identity_fn[Real](x) = x
        cube_real(x) = x * x * x
        pointwise_mul[Real, Real](cube_real, identity_fn[Real], x) = cube_real(x) * identity_fn[Real](x)
        pointwise_mul[Real, Real](cube_real, identity_fn[Real], x) = (x * x * x) * x
        quartic_real(x) = x * x * x * x
        quartic_real(x) = pointwise_mul[Real, Real](cube_real, identity_fn[Real], x)
    }
}

/// The quartic function on the reals is continuous at each point.
theorem continuous_at_quartic_real(x: Real) {
    continuous_at(quartic_real, x)
} by {
    continuous_at_cube_real(x)
    continuous_at(cube_real, x)
    identity_function_is_continuous
    continuous_imp_continuous_at(identity_fn[Real], x)
    continuous_at(identity_fn[Real], x)
    continuous_at_pointwise_mul(cube_real, identity_fn[Real], x)
    continuous_at(pointwise_mul[Real, Real](cube_real, identity_fn[Real]), x)
    quartic_real_eq_pointwise_mul_cube_identity
    continuous_at(quartic_real, x)
}

/// The quartic function on the reals is continuous.
theorem continuous_quartic_real {
    continuous(quartic_real)
} by {
    continuous_cube_real
    continuous(cube_real)
    identity_function_is_continuous
    continuous(identity_fn[Real])
    continuous_pointwise_mul(cube_real, identity_fn[Real])
    continuous(pointwise_mul[Real, Real](cube_real, identity_fn[Real]))
    quartic_real_eq_pointwise_mul_cube_identity
    continuous(quartic_real)
}
