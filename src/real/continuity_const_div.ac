from real.continuity_const_mul import const_mul_right,
    continuous_at_const_mul_right, continuous_const_mul_right
from real.continuity_base import Real, continuous, continuous_at

/// The function obtained by dividing f on the right by the fixed real constant c.
define const_div_right(f: Real -> Real, c: Real, x: Real) -> Real {
    f(x) / c
}

/// Dividing by a constant on the right agrees with multiplication by its inverse.
theorem const_div_right_eq_const_mul_right_inverse(f: Real -> Real, c: Real) {
    const_div_right(f, c) = const_mul_right(f, c.inverse)
} by {
    forall(x: Real) {
        f(x) / c = f(x) * c.inverse
        const_div_right(f, c, x) = f(x) / c
        const_mul_right(f, c.inverse, x) = f(x) * c.inverse
        const_div_right(f, c, x) = const_mul_right(f, c.inverse, x)
    }
}

/// Dividing by a fixed real constant on the right preserves continuity at a point.
theorem continuous_at_const_div_right(f: Real -> Real, c: Real, x: Real) {
    continuous_at(f, x)
    implies continuous_at(const_div_right(f, c), x)
} by {
    if continuous_at(f, x) {
        continuous_at_const_mul_right(f, c.inverse, x)
        continuous_at(const_mul_right(f, c.inverse), x)
        const_div_right_eq_const_mul_right_inverse(f, c)
        continuous_at(const_div_right(f, c), x)
    }
}

/// Dividing by a fixed real constant on the right preserves continuous functions.
theorem continuous_const_div_right(f: Real -> Real, c: Real) {
    continuous(f) implies continuous(const_div_right(f, c))
} by {
    if continuous(f) {
        continuous_const_mul_right(f, c.inverse)
        continuous(const_mul_right(f, c.inverse))
        const_div_right_eq_const_mul_right_inverse(f, c)
        continuous(const_div_right(f, c))
    }
}
