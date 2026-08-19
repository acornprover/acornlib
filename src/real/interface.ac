/// Public interface for real numbers.

from algebra.add import Add
from lte import LTE
from nat import Nat, from_nat, pow_zero, semiring_zero_pow, alt_induction, from_nat_zero, from_nat_add
from algebra.neg import Neg
from int import Int, abs
from rat import Rat, iop, finite_seq_abs_bounded, no_greatest
from algebra.zero import Zero
from data.basic.relation_basic import is_reflexive, is_transitive, is_antisymmetric
from order import PartialOrder, LinearOrder, min_eq_left_of_lt, min_eq_right_of_not_lt, max_eq_left_of_gt, max_eq_right_of_not_gt, lt_min_iff, max_lt_iff, lte_min_of_bounds, lte_max_left, lte_max_right, max_lte_of_upper_bounds, min_max_distrib_left, max_min_distrib_left, lt_of_lte_of_lt, lte_antisymm, lt_of_lt_of_lte, max_imp_gte
from lattice import Meet, Join, MeetSemilattice, JoinSemilattice, Lattice, DistribLattice
from algebra.one import One
from algebra.add_semigroup import AddSemigroup, add_fn
from algebra.add_comm_semigroup import AddCommSemigroup
from algebra.add_monoid import AddMonoid
from algebra.add_comm_monoid import AddCommMonoid
from algebra.add_group import AddGroup
from algebra.add_comm_group import AddCommGroup
from algebra.add_ordered_group import AddLeftOrderedGroup, AddOrderedGroup
from data.basic.functions import compose, flip, identity_fn
from data.basic.function_algebra import pointwise_add, pointwise_neg, pointwise_mul
from data.basic.logic import eq_true_intro, eq_true_elim
from algebra.mul import Mul
from algebra.semigroup import Semigroup, mul_fn
from algebra.monoid.monoid import Monoid
from semiring import Semiring
from algebra.ring.ring import Ring, alternating_sign, alternating_sign_zero, alternating_sign_suc
from algebra.comm_semigroup import CommSemigroup
from algebra.comm_monoid import CommMonoid
from comm_ring import CommRing, binomial_term, binomial
from list import partial, sum, map, List, map_add, sum_add, partial_scalar_mul, partial_add, partial_pointwise_eq
from order import is_monotone, is_antitone, monotone_from_forall, antitone_from_forall
from algebra.inverse import Inverse
from algebra.field.field import Field, mul_not_zero
from algebra.integral_domain import IntegralDomain
from ordered_field import OrderedField, mul_le_mul_of_nonneg_right, mul_le_mul_of_nonneg_left, mul_lt_mul_of_pos_right, inverse_on_positive_flips_inequality
from combinatorics import binom
from data.basic.set import Set, image_contains, set_image, set_image_compose, set_image_contains_witness, maps_into_set_image, universal_set_contains_eq
from data.basic.witness import choose_or_default, choose_or_default_spec, choose_or_default_unique_eq, exists_unique, exists_unique_intro

// real_base.ac
// This file contains the initial definition of real numbers, based on Dedekind cuts.
// It handles addition, negation, subtraction, and ordering.

/// True if a function on rationals partitions them into two non-empty sets.
define is_cut(f: Rat -> Bool) -> Bool {
    exists(x: Rat) {
        f(x)
    } and exists(x: Rat) {
        not f(x)
    }
}

/// True if a set is downward-closed (contains all smaller elements).
define is_lower(f: Rat -> Bool) -> Bool {
    forall(x: Rat, y: Rat) {
        f(y) and x < y implies f(x)
    }
}

theorem lower_contains_smaller(f: Rat -> Bool, x: Rat, y: Rat) {
    is_lower(f) and f(y) and x < y implies f(x)
}

/// True if x is the greatest element in the set defined by f.
define is_greatest(f: Rat -> Bool, x: Rat) -> Bool {
    f(x) and forall(y: Rat) {
        f(y) implies y <= x
    }
}

/// True if the set defined by f has a greatest element.
define has_greatest(f: Rat -> Bool) -> Bool {
    exists(x: Rat) {
        is_greatest(f, x)
    }
}

/// True if a function represents a valid Dedekind cut defining a real number.
define is_dedekind_cut(f: Rat -> Bool) -> Bool {
    is_cut(f) and is_lower(f) and not has_greatest(f)
}

// "All numbers y such that x is greater then y" is the cut that embeds x.

theorem gt_is_cut(r: Rat) {
    is_cut(r.gt)
}

theorem gt_is_lower(r: Rat) {
    is_lower(r.gt)
}

theorem gt_has_no_greatest(r: Rat) {
    not has_greatest(r.gt)
}

theorem gt_is_dedekind_cut(r: Rat) {
    is_dedekind_cut(r.gt)
}

/// Real numbers are defined by a Dedekind cut. Specifically, using the `gt_rat` function which
/// specifies which rationals they are greater than.
structure Real {
    /// True if this real number is greater than the given rational number.
    gt_rat: Rat -> Bool
} constraint {
    is_dedekind_cut(gt_rat)
}

let real_from_rat(r: Rat) -> a: Real satisfy {
    Real.new(r.gt) = Option.some(a)
}

/// Real extensionality: two real numbers are equal when they have the same cut.
theorem real_ext(a: Real, b: Real) {
    (forall(r: Rat) { a.gt_rat(r) = b.gt_rat(r) }) implies a = b
}

attributes Real {
    /// Converts a rational number to a real number.
    let from_rat = real_from_rat

    /// Real extensionality from pointwise equality of the cut.
    let ext = real_ext
}

instance Real: Zero {
    let 0: Real = Real.from_rat(Rat.0)
}

attributes Real {
    /// True if this real number is positive (greater than zero).
    define is_positive(self) -> Bool {
        self.gt_rat(Rat.0)
    }

    /// True if this real number is negative (less than zero).
    define is_negative(self) -> Bool {
        self != Real.0 and not self.is_positive
    }
}

/// The less-than-or-equal-to relation for real numbers.
instance Real: LTE {
    define lte(self, other: Real) -> Bool {
        forall(r: Rat) {
            self.gt_rat(r) implies other.gt_rat(r)
        }
    }
}

theorem lte_self(r: Real) {
    r <= r
}

theorem lte_trans(a: Real, b: Real, c: Real) {
    a <= b and b <= c implies a <= c
}

/// Transitivity with equality: if a <= b and b = c, then a <= c.
theorem lte_trans_eq(a: Real, b: Real, c: Real) {
    a <= b and b = c implies a <= c
}

theorem gt_rat_sorts(z: Real, r1: Rat, r2: Rat) {
    z.gt_rat(r1) and not z.gt_rat(r2) implies r1 <= r2
}

theorem lte_or_gte(a: Real, b: Real) {
    a <= b or b <= a
}

theorem lte_both_ways_imp_eq(a: Real, b: Real) {
    a <= b and b <= a implies a = b
}

// The real numbers form a total order.


theorem real_is_reflexive {
    is_reflexive(Real.lte)
}

theorem real_is_transitive {
    is_transitive(Real.lte)
}

theorem real_is_antisymmetric {
    is_antisymmetric(Real.lte)
}

instance Real: PartialOrder

instance Real: LinearOrder

theorem gt_imp_from_rat_gt(r1: Rat, r2: Rat) {
    r1 > r2 implies Real.from_rat(r1).gt_rat(r2)
}

theorem from_rat_gt_imp_gt(r1: Rat, r2: Rat) {
    Real.from_rat(r1).gt_rat(r2) implies r1 > r2
}

theorem not_gt_rat_self(r: Rat) {
    not Real.from_rat(r).gt_rat(r)
}

theorem zero_not_positive {
    not Real.0.is_positive
}

theorem gte_self(r: Real) {
    r >= r
}

theorem gt_rat_imp_gt_from_rat(a: Real, r: Rat) {
    a.gt_rat(r) implies a > Real.from_rat(r)
}

theorem gt_from_rat_imp_gt_rat(a: Real, r: Rat) {
    a > Real.from_rat(r) implies a.gt_rat(r)
}

theorem pos_gt_zero(a: Real) {
    a.is_positive implies a > Real.0
}

theorem gt_zero_imp_pos(a: Real) {
    a > Real.0 implies a.is_positive
}

theorem neg_lt_zero(a: Real) {
    a.is_negative implies a < Real.0
}

theorem lt_zero_imp_neg(a: Real) {
    a < Real.0 implies a.is_negative
}

// Informally, z1 + z2 > r.
define add_gt(z1: Real, z2: Real, r: Rat) -> Bool {
    exists(r1: Rat, r2: Rat) {
        z1.gt_rat(r1) and z2.gt_rat(r2) and r = r1 + r2
    }
}

theorem add_gt_symm(z1: Real, z2: Real, r: Rat) {
    add_gt(z1, z2, r) implies add_gt(z2, z1, r)
}

theorem exists_lesser_rat(z: Real) {
    exists(r: Rat) {
        z.gt_rat(r)
    }
}

theorem exists_gte_rat(z: Real) {
    exists(r: Rat) {
        not z.gt_rat(r)
    }
}

theorem add_gt_is_cut(z1: Real, z2: Real) {
    is_cut(add_gt(z1, z2))
}

theorem add_gt_is_lower(z1: Real, z2: Real) {
    is_lower(add_gt(z1, z2))
}

theorem add_gt_has_no_greatest(z1: Real, z2: Real) {
    not has_greatest(add_gt(z1, z2))
}

theorem add_gt_is_dedekind_cut(z1: Real, z2: Real) {
    is_dedekind_cut(add_gt(z1, z2))
}

let add(a: Real, b: Real) -> c: Real satisfy {
    Real.new(add_gt(a, b)) = Option.some(c)
}

/// The sum of two real numbers.
instance Real: Add {
    let add = add
}

theorem add_gt_rat(z1: Real, z2: Real, r1: Rat, r2: Rat) {
    z1.gt_rat(r1) and z2.gt_rat(r2) implies (z1 + z2).gt_rat(r1 + r2)
}

theorem add_comm(a: Real, b: Real) {
    a + b = b + a
}

theorem gt_rat_adding_three(z1: Real, z2: Real, z3: Real, q: Rat) {
    (z1 + z2 + z3).gt_rat(q) implies exists(r1: Rat, r2: Rat, r3: Rat) {
        z1.gt_rat(r1) and z2.gt_rat(r2) and z3.gt_rat(r3) and q = r1 + r2 + r3
    }
}

theorem gt_rat_adding_three_converse(z1: Real, z2: Real, z3: Real, q: Rat) {
    exists(r1: Rat, r2: Rat, r3: Rat) {
        z1.gt_rat(r1) and z2.gt_rat(r2) and z3.gt_rat(r3) and q = r1 + r2 + r3
    } implies (z1 + z2 + z3).gt_rat(q)
}

theorem add_assoc(a: Real, b: Real, c: Real) {
    a + b + c = a + (b + c)
}

theorem gt_imp_not_lte(a: Real, b: Real) {
    a > b implies not a <= b
}

theorem not_lte_imp_gt(a: Real, b: Real) {
    not a <= b implies a > b
}

theorem gte_imp_not_lt(a: Real, b: Real) {
    a >= b implies not a < b
}

theorem not_lt_imp_gte(a: Real, b: Real) {
    not a < b implies a >= b
}

theorem rat_separating(a: Real, b: Real) {
    a < b implies exists(r: Rat) {
        b.gt_rat(r) and not a.gt_rat(r)
    }
}

theorem rat_between_rat_and_real(z: Real, r1: Rat) {
    z.gt_rat(r1) implies exists(r2: Rat) {
        z.gt_rat(r2) and r1 < r2
    }
}

// Strict version of rat_separating.
theorem rat_between_reals(a: Real, b: Real) {
    a < b implies exists(r: Rat) {
        a < Real.from_rat(r) and Real.from_rat(r) < b
    }
}

theorem rat_between_reals_gt(a: Real, b: Real) {
    a > b implies
    exists(r: Rat) {
        a > Real.from_rat(r) and Real.from_rat(r) > b
    }
}

theorem add_gt_trans(z1: Real, z2: Real, r1: Rat, r2: Rat) {
    add_gt(z1, z2, r1) and r1 > r2 implies add_gt(z1, z2, r2)
}

theorem add_gt_imp_gt_from_rat(z1: Real, z2: Real, r: Rat) {
    add_gt(z1, z2, r) implies z1 + z2 > Real.from_rat(r)
}

theorem gt_from_rat_imp_add_gt(z1: Real, z2: Real, r: Rat) {
    z1 + z2 > Real.from_rat(r) implies add_gt(z1, z2, r)
}

theorem lt_lte_trans(a: Real, b: Real, c: Real) {
    a < b and b <= c implies a < c
}

theorem lte_lt_trans(a: Real, b: Real, c: Real) {
    a <= b and b < c implies a < c
}

theorem lt_trans(a: Real, b: Real, c: Real) {
    a < b and b < c implies a < c
}

theorem add_gt_from_rat_imp_rat_add_gt(p: Rat, q: Rat, r: Rat) {
    add_gt(Real.from_rat(p), Real.from_rat(q), r) implies p + q > r
}

theorem rat_add_gt_imp_add_gt_from_rat(p: Rat, q: Rat, r: Rat) {
    p + q > r implies add_gt(Real.from_rat(p), Real.from_rat(q), r)
}

theorem add_from_rat(p: Rat, q: Rat) {
    Real.from_rat(p) + Real.from_rat(q) = Real.from_rat(p + q)
}

theorem lte_add_right(a: Real, b: Real, c: Real) {
    a <= b implies a + c <= b + c
}

theorem lte_add_left(a: Real, b: Real, c: Real) {
    a <= b implies c + a <= c + b
}

/// Adding two inequalities preserves the inequality.
theorem add_lte_add(a: Real, b: Real, c: Real, d: Real) {
    a <= b and c <= d implies a + c <= b + d
}

theorem lt_add_converse(a: Real, b: Real, c: Real) {
    a + c < b + c implies a < b
}



theorem add_zero_right(a: Real) {
    a + Real.0 = a
}

theorem add_zero_left(a: Real) {
    Real.0 + a = a
}

theorem from_rat_maintains_lt(p: Rat, q: Rat) {
    p < q implies Real.from_rat(p) < Real.from_rat(q)
}

theorem add_lt_lt(a: Real, b: Real, c: Real, d: Real) {
    a < b and c < d implies a + c < b + d
}

theorem lte_some_rat(a: Real) {
    exists(r: Rat) {
        a <= Real.from_rat(r)
    }
}

theorem lt_some_rat(a: Real) {
    exists(r: Rat) {
        a < Real.from_rat(r)
    }
}

theorem gt_some_rat(a: Real) {
    exists(r: Rat) {
        a > Real.from_rat(r)
    }
}

attributes Real {
    /// Converts an integer to a real number.
    let from_int = function(n: Int) {
        Real.from_rat(Rat.from_int(n))
    }
}

theorem from_rat_maintains_lte(p: Rat, q: Rat) {
    p <= q implies Real.from_rat(p) <= Real.from_rat(q)
}

theorem gt_some_int(a: Real) {
    exists(n: Int) {
        a > Real.from_int(n)
    }
}

theorem lt_some_int(a: Real) {
    exists(n: Int) {
        a < Real.from_int(n)
    }
}

theorem real_neg_imp_rat_neg(r: Rat) {
    Real.from_rat(r).is_negative implies r.is_negative
}

theorem lt_some_int_cancel(m: Int, n: Int) {
    Real.from_int(m) < Real.from_int(n) implies m < n
}

theorem floor_exists(a: Real) {
    exists(n: Int) {
        Real.from_int(n) <= a and a < Real.from_int(n + Int.1)
    }
}

let floor(a: Real) -> n: Int satisfy {
    Real.from_int(n) <= a and a < Real.from_int(n + Int.1)
}

instance Real: One {
    let 1: Real = Real.from_rat(Rat.1)
}

attributes Real {
    /// One half (1/2).
    let one_half: Real = Real.from_rat(Rat.2.inverse)
}

theorem one_half_positive {
    Real.0 < Real.one_half
}

theorem one_half_plus_one_half {
    Real.one_half + Real.one_half = Real.1
}

theorem add_from_int(m: Int, n: Int) {
    Real.from_int(m) + Real.from_int(n) = Real.from_int(m + n)
}

theorem lt_add_one(a: Real) {
    a < a + Real.1
}

theorem lt_add_pos_int(a: Real, n: Int) {
    n.is_positive implies a < a + Real.from_int(n)
}

theorem lt_add_pos_rat(a: Real, r: Rat) {
    r.is_positive implies a < a + Real.from_rat(r)
}

theorem lt_add_pos(a: Real, b: Real) {
    b.is_positive implies a < a + b
}

theorem lt_add_rat_right(a: Real, b: Real, r: Rat) {
    a < b implies a + Real.from_rat(r) < b + Real.from_rat(r)
}

theorem lt_add_rat_left(a: Real, b: Real, r: Rat) {
    a < b implies Real.from_rat(r) + a < Real.from_rat(r) + b
}

theorem add_rat_eps_between(a: Real, b: Real) {
    a < b implies exists(eps: Rat) {
        eps.is_positive and a + Real.from_rat(eps) < b
    }
}

theorem lt_add_right(a: Real, b: Real, c: Real) {
    a < b implies a + c < b + c
}

theorem lt_add_left(a: Real, b: Real, c: Real) {
    a < b implies c + a < c + b
}

theorem lt_add_cancel_right(a: Real, b: Real, c: Real) {
    a + c = b + c implies a = b
}

theorem lt_add_cancel_left(a: Real, b: Real, c: Real) {
    c + a = c + b implies a = b
}

// Whether -a > r. Which is just a < -r.
define neg_gt(a: Real, r: Rat) -> Bool {
    a < Real.from_rat(-r)
}

theorem lower_rat(a: Real) {
    exists(r: Rat) {
        Real.from_rat(r) < a
    }
}

theorem neg_gt_is_cut(a: Real) {
    is_cut(neg_gt(a))
}

theorem neg_gt_is_lower(a: Real) {
    is_lower(neg_gt(a))
}

theorem neg_gt_has_no_greatest(a: Real) {
    not has_greatest(neg_gt(a))
}

theorem neg_gt_is_dedekind_cut(a: Real) {
    is_dedekind_cut(neg_gt(a))
}

let neg(a: Real) -> b: Real satisfy {
    Real.new(neg_gt((a))) = Option.some(b)
}

/// The negative of this real number.
instance Real: Neg {
    let neg = neg
}

theorem lt_rat_neg(a: Real, r: Rat) {
    a < Real.from_rat(-r) implies -a > Real.from_rat(r)
}

theorem neg_from_rat(r: Rat) {
    -Real.from_rat(r) = Real.from_rat(-r)
}

theorem neg_gt_rat(a: Real, r: Rat) {
    -a > Real.from_rat(r) implies a < Real.from_rat(-r)
}

theorem gt_rat_neg(a: Real, r: Rat) {
    a > Real.from_rat(-r) implies -a < Real.from_rat(r)
}

theorem neg_lt_rat(a: Real, r: Rat) {
    -a < Real.from_rat(r) implies a > Real.from_rat(-r)
}

theorem lt_swap_neg(a: Real, b: Real) {
    a < b implies -b < -a
}

theorem lt_neg_swap_neg(a: Real, b: Real) {
    a < -b implies b < -a
}

theorem neg_neg(a: Real) {
    -(-a) = a
}

theorem neg_lt_swap_neg(a: Real, b: Real) {
    -a < b implies -b < a
}

theorem neg_lt_neg_swap_neg(a: Real, b: Real) {
    -a < -b implies b < a
}

theorem neg_zero {
    -Real.0 = Real.0
}

theorem neg_pos_is_neg(a: Real) {
    a.is_positive implies (-a).is_negative
}

theorem neg_neg_is_pos(a: Real) {
    (-a).is_negative implies a.is_positive
}

theorem gt_add_neg(a: Real, b: Real) {
    b.is_negative implies a > a + b
}

theorem rat_window(a: Real, eps: Rat) {
    eps.is_positive implies exists(r: Rat) {
        Real.from_rat(r) < a and a < Real.from_rat(r + eps)
    }
}

theorem add_neg_eq_zero(a: Real) {
    a + -a = Real.0
}

theorem neg_distrib(a: Real, b: Real) {
    -(a + b) = -a + -b
}

// Additive algebraic structure.


instance Real: AddSemigroup

instance Real: AddCommSemigroup

instance Real: AddMonoid

instance Real: AddCommMonoid

instance Real: AddGroup

instance Real: AddCommGroup

instance Real: AddLeftOrderedGroup

instance Real: AddOrderedGroup

attributes Real {
    /// The absolute value of this real number.
    define abs(self) -> Real {
        if self.is_negative {
            -self
        } else {
            self
        }
    }

    /// The sign of this real number as a unit value (`-1` for negative, `1` for non-negative).
    define unit_sign(self) -> Real {
        if self.is_negative {
            -Real.1
        } else {
            Real.1
        }
    }
}

theorem pos_imp_eq_abs(a: Real) {
    a.is_positive implies a = a.abs
}

theorem sub_cancels(a: Real, b: Real) {
    a + b - b = a
}

theorem sub_moves_sides(a: Real, b: Real, c: Real) {
    a + b = c implies a = c - b
}

theorem lte_abs(a: Real) {
    a <= a.abs
}

theorem abs_neg(a: Real) {
    (-a).abs = a.abs
}

theorem abs_not_neg(a: Real) {
    not a.abs.is_negative
}

theorem min_pos_pos(a: Real, b: Real) {
    a.is_positive and b.is_positive
    implies
    a.min(b).is_positive
}





theorem min_lte_left(a: Real, b: Real) {
    a.min(b) <= a
}

theorem min_lte_right(a: Real, b: Real) {
    a.min(b) <= b
}



theorem lt_both_imp_lt_min(a: Real, b: Real, c: Real) {
    a < b and a < c implies a < b.min(c)
}

theorem lt_min_imp_lt_left(a: Real, b: Real, c: Real) {
    a < b.min(c) implies a < b
}

theorem lt_min_imp_lt_right(a: Real, b: Real, c: Real) {
    a < b.min(c) implies a < c
}

theorem gt_both_imp_gt_max(a: Real, b: Real, c: Real) {
    a > b and a > c implies a > b.max(c)
}

theorem gt_max_imp_gt_left(a: Real, b: Real, c: Real) {
    a > b.max(c) implies a > b
}

theorem gt_max_imp_gt_right(a: Real, b: Real, c: Real) {
    a > b.max(c) implies a > c
}

instance Real: Meet {
    define meet(self, other: Real) -> Real {
        self.min(other)
    }
}

instance Real: Join {
    define join(self, other: Real) -> Real {
        self.max(other)
    }
}

theorem real_meet_lte_left(a: Real, b: Real) {
    a.meet(b) <= a
}

theorem real_meet_lte_right(a: Real, b: Real) {
    a.meet(b) <= b
}

theorem real_lte_meet_of_bounds(c: Real, a: Real, b: Real) {
    c <= a and c <= b implies c <= a.meet(b)
}

instance Real: MeetSemilattice

theorem real_lte_join_left(a: Real, b: Real) {
    a <= a.join(b)
}

theorem real_lte_join_right(a: Real, b: Real) {
    b <= a.join(b)
}

theorem real_join_lte_of_bounds(a: Real, b: Real, c: Real) {
    a <= c and b <= c implies a.join(b) <= c
}

instance Real: JoinSemilattice

instance Real: Lattice

theorem real_meet_join_distrib_left(a: Real, b: Real, c: Real) {
    a.meet(b.join(c)) = a.meet(b).join(a.meet(c))
}

theorem real_join_meet_distrib_left(a: Real, b: Real, c: Real) {
    a.join(b.meet(c)) = a.join(b).meet(a.join(c))
}

instance Real: DistribLattice

theorem rat_dual_upper_bound(a: Real, b: Real) {
    exists(r: Rat) {
        a < Real.from_rat(r) and b < Real.from_rat(r)
    }
}

theorem abs_gte_zero(a: Real) {
    a.abs >= Real.0
}

theorem abs_from_rat(p: Rat) {
    Real.from_rat(p).abs = Real.from_rat(p.abs)
}

attributes Real {
    /// True if this real number is within `eps` of the other real number.
    define is_close(self, other: Real, eps: Real) -> Bool {
        (self - other).abs < eps
    }
}

theorem close_imp_eps_pos(a: Real, b: Real, eps: Real) {
    a.is_close(b, eps) implies eps.is_positive
}

theorem close_comm(a: Real, b: Real, eps: Real) {
    a.is_close(b, eps) implies b.is_close(a, eps)
}

theorem close_imp_bounds(a: Real, b: Real, eps: Real) {
    a.is_close(b, eps) implies b - eps < a and a < b + eps and a > b - eps and b < a + eps
}

theorem bounds_imp_close(a: Real, b: Real, eps: Real) {
    b - eps < a and a < b + eps implies a.is_close(b, eps)
}

theorem sum_bounds_imp_close(a: Real, b: Real, eps: Real) {
    b < a + eps and a < b + eps implies a.is_close(b, eps)
}

theorem self_close(a: Real, eps: Real) {
    eps.is_positive implies a.is_close(a, eps)
}

// Closeness is preserved in the real<->rat conversion.
theorem close_rats_imp_close_reals(q: Rat, r: Rat, eps: Rat) {
    q.is_close(r, eps) implies Real.from_rat(q).is_close(Real.from_rat(r), Real.from_rat(eps))
}

theorem close_reals_imp_close_rats(q: Rat, r: Rat, eps: Rat) {
    Real.from_rat(q).is_close(Real.from_rat(r), Real.from_rat(eps))
    implies q.is_close(r, eps)
}

// Every real can be approximated by a rational.
theorem rat_approx_exists(x: Real, eps: Rat) {
    eps.is_positive implies exists(r: Rat) {
        x.is_close(Real.from_rat(r), Real.from_rat(eps))
    }
}

// Every real has a rational that is a distant upper bound.
theorem rat_upper(x: Real, eps: Rat) {
    eps.is_positive implies exists(r: Rat) {
        not x.is_close(Real.from_rat(r), Real.from_rat(eps)) and x < Real.from_rat(r)
    }
}

// Every real has a rational that is a distant lower bound.
theorem rat_lower(x: Real, eps: Rat) {
    eps.is_positive implies exists(r: Rat) {
        not x.is_close(Real.from_rat(r), Real.from_rat(eps)) and Real.from_rat(r) < x
    }
}

// Every intersecting pair of intervals has a rational number in it.
theorem rat_intersect(a: Real, b: Real, c: Real, d: Real, e: Real) {
    a.is_close(b, c) and a.is_close(d, e)
    implies
    exists(r: Rat) {
        Real.from_rat(r).is_close(b, c) and Real.from_rat(r).is_close(d, e)
    }
}

theorem swap_minus_plus(a: Real, b: Real, c: Real) {
    a + b - c = a - c + b
}

// real_seq.ac
/// The bound on the end of a sequence that's part of the Cauchy convergence condition.
/// Separating it out like this makes convergence a bit clearer to prove.
define cauchy_bound(q: Nat -> Real, n: Nat, eps: Real) -> Bool {
    forall(i: Nat, j: Nat) {
        n <= i and n <= j implies q(i).is_close(q(j), eps)
    }
}

/// The Cauchy condition for the convergence of a sequence.
define converges(q: Nat -> Real) -> Bool {
    forall(eps: Real) {
        eps.is_positive implies exists(n: Nat) {
            cauchy_bound(q, n, eps)
        }
    }
}

/// lb is eventually a lower bound of this sequence.
define eventual_lb(q: Nat -> Real, lb: Real) -> Bool {
    exists(n: Nat) {
        forall(i: Nat) {
            n <= i implies lb <= q(i)
        }
    }
}

/// For demonstrating the equivalent of Cauchy sequences and Dedekind cuts.
define cauchy_gt_rat(q: Nat -> Real, r: Rat) -> Bool {
    exists(lb: Real) {
        lb > Real.from_rat(r) and eventual_lb(q, lb)
    }
}

/// limit is defined with a "default" so that it's zero for things that don't converge.
let limit(q: Nat -> Real) -> lim: Real satisfy {
    if converges(q) {
        Real.new(cauchy_gt_rat(q)) = Option.some(lim)
    } else {
        lim = Real.0
    }
}

// The other direction of the lower bound.
// We can prove basically the same stuff here.
define eventual_ub(q: Nat -> Real, ub: Real) -> Bool {
    exists(n: Nat) {
        forall(i: Nat) {
            n <= i implies q(i) <= ub
        }
    }
}

define eventual_eq(q: Nat -> Real, a: Real) -> Bool {
    exists(n: Nat) {
        forall(i: Nat) {
            n <= i implies q(i) = a
        }
    }
}

theorem lt_imp_minus_pos(a: Real, b: Real) {
    a < b implies (b - a).is_positive
}

theorem from_rat_pos(r: Rat) {
    r.is_positive implies Real.from_rat(r).is_positive
}

// The bound on the end of a sequence that's part of the Weierstress convergence definition.
// Separating it out like this makes convergence a bit clearer to prove.
define tail_bound(q: Nat -> Real, a: Real, n: Nat, eps: Real) -> Bool {
    forall(i: Nat) {
        n <= i implies q(i).is_close(a, eps)
    }
}

theorem tail_bound_implies_is_close(q: Nat -> Real, a: Real, n: Nat, eps: Real, i: Nat) {
    tail_bound(q, a, n, eps) and n <= i implies q(i).is_close(a, eps)
}

// converges_to is the Weierstrass definition.
define converges_to(q: Nat -> Real, a: Real) -> Bool {
    forall(eps: Real) {
        eps.is_positive implies exists(n: Nat) {
            tail_bound(q, a, n, eps)
        }
    }
}

theorem eps_lt_half(a: Real) {
    a.is_positive implies exists(b: Real) {
        b.is_positive and b + b < a
    }
}

// Prove our two definitions of convergence are equivalent.

theorem converges_to_imp_converges(q: Nat -> Real, a: Real) {
    converges_to(q, a) implies converges(q)
}

theorem converges_imp_converges_to(q: Nat -> Real) {
    converges(q) implies converges_to(q, limit(q))
}

theorem converges_to_unique(q: Nat -> Real, a: Real, b: Real) {
    converges_to(q, a) and converges_to(q, b) implies a = b
}

// Lifting a sequence of rationals to a sequence of reals
define lift_seq(q: Nat -> Rat) -> Nat -> Real {
    compose(Real.from_rat, q)
}

let rat_approx(x: Real, eps: Rat) -> r: Rat satisfy {
    eps.is_positive implies
    x.is_close(Real.from_rat(r), Real.from_rat(eps))
}

// A sequence of rationals that approximates a real number.
// Chosen to be easy to calculate with.
define rat_seq(x: Real, n: Nat) -> Rat {
    rat_approx(x, iop(n))
}

define add_seq(a: Nat -> Real, b: Nat -> Real, n: Nat) -> Real {
    a(n) + b(n)
}

theorem add_close(a1: Real, b1: Real, a2: Real, b2: Real, a_eps: Real, b_eps: Real) {
    a1.is_close(a2, a_eps) and b1.is_close(b2, b_eps)
    implies
    (a1 + b1).is_close(a2 + b2, a_eps + b_eps)
}

// When two sequences get close to each other eventually.
define seq_close(a: Nat -> Real, b: Nat -> Real, eps: Real) -> Bool {
    exists(n: Nat) {
        forall(i: Nat) {
            n <= i implies a(i).is_close(b(i), eps)
        }
    }
}

theorem only_abs_zero_eq_zero(x: Real) {
    x.abs = Real.0 implies x = Real.0
}

theorem close_and_lt_imp_close(x: Real, y: Real, eps1: Real, eps2: Real) {
    x.is_close(y, eps1) and eps1 < eps2 implies x.is_close(y, eps2)
}

theorem sub_either_order(a: Real, b: Real, c: Real) {
    a - b - c = a - c - b
}

theorem sub_both_eq_sub_add(a: Real, b: Real, c: Real) {
    a - b - c = a - (b + c)
}

theorem sub_zero_imp_eq(x: Real, y: Real) {
    x - y = Real.0 implies x = y
}

theorem diff_pos_imp_lt(a: Real, b: Real) {
    (b - a).is_positive implies a < b
}

define limit_rat(a: Nat -> Rat) -> Real {
    limit(lift_seq(a))
}

// Equivalence classes between Cauchy sequences.
define eq_seq(a: Nat -> Rat, b: Nat -> Rat) -> Bool {
    converges(lift_seq(a)) and converges_to(lift_seq(b), limit_rat(a))
}

define add_rat_seq(a: Nat -> Rat, b: Nat -> Rat, n: Nat) -> Rat {
    a(n) + b(n)
}

define mul_rat_seq(a: Nat -> Rat, b: Nat -> Rat, n: Nat) -> Rat {
    a(n) * b(n)
}

define neg_rat_seq(a: Nat -> Rat, n: Nat) -> Rat {
    -a(n)
}

theorem neg_is_close(x: Real, y: Real, eps: Real) {
    x.is_close(y, eps) implies
    (-x).is_close(-y, eps)
}

define sub_rat_seq(a: Nat -> Rat, b: Nat -> Rat, n: Nat) -> Rat {
    a(n) - b(n)
}

define const_rat_seq(a: Rat, n: Nat) -> Rat {
    a
}

let zero_rat_seq = const_rat_seq(Rat.0)

// cauchy_criterion.ac
/// The Cauchy criterion for real-valued sequences.

/// The Cauchy tail condition at a fixed tolerance.
define is_cauchy_seq_at(a: Nat -> Real, eps: Real) -> Bool {
    eps.is_positive implies exists(n: Nat) {
        forall(i: Nat, j: Nat) {
            n <= i and n <= j implies a(i).is_close(a(j), eps)
        }
    }
}

/// True if a is a Cauchy sequence: for every positive epsilon, terms past
/// some index are within epsilon of one another.
define is_cauchy_seq(a: Nat -> Real) -> Bool {
    forall(eps: Real) {
        is_cauchy_seq_at(a, eps)
    }
}

/// A Cauchy real sequence converges to its limit.
theorem cauchy_imp_converges_to_limit(a: Nat -> Real) {
    is_cauchy_seq(a) implies converges_to(a, limit(a))
}

/// A Cauchy real sequence has some real limit.
theorem cauchy_imp_exists_limit(a: Nat -> Real) {
    is_cauchy_seq(a) implies exists(x: Real) {
        converges_to(a, x)
    }
}

// real_ring.ac
/// The product of two real numbers.
instance Real: Mul {
    define mul(self, other: Real) -> Real {
        limit_rat(mul_rat_seq(rat_seq(self), rat_seq(other)))
    }
}

theorem real_mul_comm(x: Real, y: Real) {
    x * y = y * x
}

theorem mul3_as_seq(x: Real, y: Real, z: Real) {
    x * y * z = limit_rat(mul_rat_seq(mul_rat_seq(rat_seq(x), rat_seq(y)), rat_seq(z)))
}

theorem mul_seq_assoc(a: Nat -> Rat, b: Nat -> Rat, c: Nat -> Rat) {
    mul_rat_seq(mul_rat_seq(a, b), c) = mul_rat_seq(a, mul_rat_seq(b, c))
}

theorem mul_assoc(x: Real, y: Real, z: Real) {
    x * (y * z) = (x * y) * z
}

theorem limit_rat_rat_seq(x: Real) {
    x = limit_rat(rat_seq(x))
}

theorem limit_definition_of_add(x: Real, y: Real) {
    x + y = limit_rat(add_rat_seq(rat_seq(x), rat_seq(y)))
}

theorem eq_seq_rat_seq_limit(a: Nat -> Rat) {
    converges(lift_seq(a)) implies
    eq_seq(a, rat_seq(limit_rat(a)))
}

theorem eq_seq_imp_limit_rat_eq(a: Nat -> Rat, b: Nat -> Rat) {
    eq_seq(a, b) implies
    limit_rat(a) = limit_rat(b)
}

theorem limit_rat_add(a: Nat -> Rat, b: Nat -> Rat) {
    converges(lift_seq(a)) and converges(lift_seq(b))
    implies
    limit_rat(add_rat_seq(a, b)) = limit_rat(a) + limit_rat(b)
}

theorem mul_distrib_right(x: Real, y: Real, z: Real) {
    x * (y + z) = x * y + x * z
}

theorem mul_distrib_left(x: Real, y: Real, z: Real) {
    (x + y) * z = x * z + y * z
}

theorem limit_rat_mul_rat_seq(a: Nat -> Rat, b: Nat -> Rat) {
    converges(lift_seq(a)) and converges(lift_seq(b))
    implies
    limit_rat(mul_rat_seq(a, b)) = limit_rat(a) * limit_rat(b)
}

theorem limit_rat_const_rat_seq(r: Rat) {
    limit_rat(const_rat_seq(r)) = Real.from_rat(r)
}

theorem mul_one_right(x: Real) {
    x * Real.1 = x
}

theorem mul_one_left(x: Real) {
    Real.1 * x = x
}

theorem mul_zero_right(x: Real) {
    x * Real.0 = Real.0
}

theorem mul_zero_left(x: Real) {
    Real.0 * x = Real.0
}

theorem mul_neg_one_left(x: Real) {
    -Real.1 * x = -x
}

theorem mul_neg_one_right(x: Real) {
    x * -Real.1 = -x
}

theorem mul_neg_left(x: Real, y: Real) {
    -x * y = -(x * y)
}

theorem mul_neg_right(x: Real, y: Real) {
    x * -y = -(x * y)
}

theorem mul_sub_distrib_right(x: Real, y: Real, z: Real) {
    x * (y - z) = x * y - x * z
}

theorem mul_sub_distrib_left(x: Real, y: Real, z: Real) {
    (x - y) * z = x * z - y * z
}

theorem real_lte_imp_rat_lte(p: Rat, q: Rat) {
    Real.from_rat(p) <= Real.from_rat(q) implies
    p <= q
}

theorem pos_lte_imp_pos(x: Real, y: Real) {
    x.is_positive and x <= y implies
    y.is_positive
}

theorem mul_pos_pos(x: Real, y: Real) {
    x.is_positive and y.is_positive
    implies
    (x * y).is_positive
}

theorem mul_neg_pos(x: Real, y: Real) {
    x.is_negative and y.is_positive
    implies
    (x * y).is_negative
}

theorem mul_pos_neg(x: Real, y: Real) {
    x.is_positive and y.is_negative
    implies
    (x * y).is_negative
}

theorem mul_neg_neg(x: Real, y: Real) {
    x.is_negative and y.is_negative
    implies
    (x * y).is_positive
}

theorem mul_nonneg_nonneg(x: Real, y: Real) {
    not x.is_negative and not y.is_negative
    implies
    not (x * y).is_negative
}

/// Product of nonnegative numbers is nonnegative.
theorem mul_nonneg(a: Real, b: Real) {
    a >= Real.0 and b >= Real.0 implies a * b >= Real.0
}

theorem square_nonneg(x: Real) {
    x * x >= Real.0
}

theorem lt_mul_pos_right(x: Real, y: Real, z: Real) {
    z.is_positive and x < y implies
    x * z < y * z
}

theorem lt_mul_pos_left(x: Real, y: Real, z: Real) {
    z.is_positive and x < y implies
    z * x < z * y
}

theorem lte_mul_nonneg_right(x: Real, y: Real, z: Real) {
    not z.is_negative and x <= y implies
    x * z <= y * z
}

theorem lte_mul_nonneg_left(x: Real, y: Real, z: Real) {
    not z.is_negative and x <= y implies
    z * x <= z * y
}

/// Multiplication is monotone for nonnegative reals.
theorem mul_le_mul_nonneg(a: Real, b: Real, c: Real, d: Real) {
    a >= Real.0 and b >= Real.0 and c >= Real.0 and d >= Real.0 and a <= c and b <= d
    implies
    a * b <= c * d
}

theorem lt_mul_neg_right(x: Real, y: Real, z: Real) {
    z.is_negative and x < y implies
    y * z < x * z
}

theorem lt_mul_neg_left(x: Real, y: Real, z: Real) {
    z.is_negative and x < y implies
    z * y < z * x
}

theorem add_neg_neg(x: Real, y: Real) {
    x.is_negative and y.is_negative implies (x + y).is_negative
}

theorem zero_lte_imp_non_neg(x: Real) {
    Real.0 <= x implies not x.is_negative
}

theorem non_neg_imp_zero_lte(x: Real) {
    not x.is_negative implies Real.0 <= x
}

theorem mul_from_rat(a: Rat, b: Rat) {
    Real.from_rat(a) * Real.from_rat(b) = Real.from_rat(a * b)
}

theorem gt_pos_is_pos(a: Real, b: Real) {
    a.is_positive and a < b implies b.is_positive
}

theorem exists_small_mul(a: Real, b: Real) {
    not a.is_negative and b.is_positive implies
    exists(c: Real) {
        c.is_positive and a * c < b
    }
}

theorem exists_small_mul_variant(a: Real, b: Real) {
    b.is_positive implies
    exists(c: Real) {
        c.is_positive and a.abs * c < b
    }
}

theorem exists_small_mul_variant_2(a: Real, b: Real) {
    a.is_positive and b.is_positive implies
    exists(c: Real) {
        c.is_positive and c * a < b
    }
}

theorem mul_abs(a: Real, b: Real) {
    a.abs * b.abs = (a * b).abs
}

theorem square_zero_imp_zero(a: Real) {
    a * a = Real.0 implies a = Real.0
}

// Multiplicative algebraic structure.


instance Real: Semigroup

instance Real: Monoid

instance Real: Semiring

instance Real: Ring

instance Real: CommSemigroup

instance Real: CommMonoid

instance Real: CommRing

// Relationship between from_nat and from_rat


/// Rat.from_nat preserves addition.
theorem rat_from_nat_add(m: Nat, n: Nat) {
    Rat.from_nat(m + n) = Rat.from_nat(m) + Rat.from_nat(n)
}

/// The embedding of natural numbers into reals via from_nat equals the composition
/// of Rat.from_nat and Real.from_rat.
theorem from_nat_is_from_rat(n: Nat) {
    from_nat[Real](n) = Real.from_rat(Rat.from_nat(n))
}

// real_series.ac
numerals Real

// seq_lte is whether every element of the sequence is lte.
define seq_lte(a: Nat -> Real, b: Nat -> Real) -> Bool {
    forall(n: Nat) {
        a(n) <= b(n)
    }
}

/// Product function for double sums.
/// Creates the outer product of two sequences: prod_fn(a, b)(i, j) = a(i) * b(j).
define prod_fn(a: Nat -> Real, b: Nat -> Real) -> ((Nat, Nat) -> Real) {
    function(i: Nat, j: Nat) { a(i) * b(j) }
}

theorem partial_suc(a: Nat -> Real, n: Nat) {
    partial(a, n.suc) = partial(a, n) + a(n)
}

// Theorem: If seq_lte(a, b), then their partials also obey seq_lte.
theorem partial_seq_lte(a: Nat -> Real, b: Nat -> Real) {
    seq_lte(a, b) implies seq_lte(partial(a), partial(b))
}

define is_upper_bound(a: Nat -> Real, b: Real) -> Bool {
    forall(n: Nat) {
        a(n) <= b
    }
}

define is_lower_bound(a: Nat -> Real, b: Real) -> Bool {
    forall(n: Nat) {
        b <= a(n)
    }
}

/// True if the sequence never decreases.
define is_increasing(a: Nat -> Real) -> Bool {
    forall(n: Nat) {
        a(n) <= a(n.suc)
    }
}

/// An increasing sequence that is bounded above converges.
///
/// The monotone convergence principle.
theorem monotone_convergence_principle(a: Nat -> Real, b: Real) {
    is_increasing(a) and is_upper_bound(a, b) implies converges(a)
}

/// The limit of a convergent increasing sequence bounds every term.
theorem increasing_convergent_bounded_by_limit(a: Nat -> Real) {
    is_increasing(a) and converges(a) implies is_upper_bound(a, limit(a))
}

/// A convergent sequence eventually below a bound has its limit below that bound.
theorem ub_imp_limit_lte(q: Nat -> Real, ub: Real) {
    converges(q) and eventual_ub(q, ub) implies limit(q) <= ub
}


theorem triangle_ineq(a: Real, b: Real) {
    (a + b).abs <= a.abs + b.abs
}

/// Partial sums of nonnegative sequences are nonnegative.
theorem partial_nonneg(f: Nat -> Real, n: Nat) {
    is_lower_bound(f, Real.0) implies partial(f, n) >= Real.0
}

define mul_seq(a: Real, b: Nat -> Real, n: Nat) -> Real {
    a * b(n)
}

theorem mul_seq_one(a: Nat -> Real) {
    mul_seq(Real.1, a) = a
}

// real_field.ac
// This file defines real division and proves theorems about it.

define recip_rat_seq(a: Nat -> Rat, n: Nat) -> Rat {
    a(n).inverse
}

theorem neg_recip_rat_seq(a: Nat -> Rat) {
    neg_rat_seq(recip_rat_seq(a)) = recip_rat_seq(neg_rat_seq(a))
}

// Division works for positive reals
theorem recip_rat_seq_pos_converges(a: Nat -> Rat, b: Real) {
    converges_to(lift_seq(a), b) and b.is_positive
    implies converges(lift_seq(recip_rat_seq(a)))
}

// Division works for all nonzero reals
theorem recip_rat_seq_converges(a: Nat -> Rat, b: Real) {
    converges_to(lift_seq(a), b) and b != Real.0
    implies converges(lift_seq(recip_rat_seq(a)))
}

/// The inverse of this real number (`1/x`). For zero, returns zero.
instance Real: Inverse {
    define inverse(self) -> Real {
        if self = Real.0 {
            Real.0
        } else {
            limit_rat(recip_rat_seq(rat_seq(self)))
        }
    }
}

define eventually_nonzero(a: Nat -> Real) -> Bool {
    exists(n: Nat) {
        forall(i: Nat) {
            n <= i implies a(i) != Real.0
        }
    }
}

theorem pos_imp_eventually_nonzero(a: Nat -> Real) {
    converges(a) and limit(a).is_positive
    implies eventually_nonzero(a)
}

theorem nonzero_imp_eventually_nonzero(a: Nat -> Real) {
    converges(a) and limit(a) != Real.0
    implies eventually_nonzero(a)
}

theorem mul_inverse(a: Real) {
    a != Real.0 implies a * a.inverse = Real.1
}

theorem zero_inverse {
    Real.0.inverse = Real.0
}

/// The rational-to-real embedding preserves nonzero inverses.
theorem real_from_rat_inverse(p: Rat) {
    p != Rat.0 implies Real.from_rat(p.inverse) = Real.from_rat(p).inverse
}

theorem zero_is_different_than_one {
    Real.0 != Real.1
}

instance Real: Field

theorem real_no_zero_divisors(a: Real, b: Real) {
    a * b = Real.0 implies a = Real.0 or b = Real.0
}

instance Real: IntegralDomain

instance Real: OrderedField

attributes Real {
    /// The quotient of two real numbers (`self/other`).
    define div(self, other: Real) -> Real {
        self * other.inverse
    }
}

theorem mul_left_cancel(a: Real, b: Real, c: Real) {
    a = b * c and b != Real.0 implies a / b = c
}

/// If a * b <= c and b > 0, then a <= c / b.
theorem div_le_of_mul_le(a: Real, b: Real, c: Real) {
    a * b <= c and b > Real.0
    implies
    a <= c / b
}

/// If a <= b and c > 0, then a * c <= b * c.
theorem mul_le_mul_pos_right(a: Real, b: Real, c: Real) {
    a <= b and c > Real.0
    implies
    a * c <= b * c
}

/// If a <= b and c > 0, then c * a <= c * b.
theorem mul_le_mul_pos_left(a: Real, b: Real, c: Real) {
    a <= b and c > Real.0
    implies
    c * a <= c * b
}

/// If a / b < c and b > 0, then a < b * c.
theorem div_lt_mul_pos(a: Real, b: Real, c: Real) {
    a / b < c and b > Real.0
    implies
    a < b * c
}

/// If c > 0, then c * (b / c) = b.
theorem mul_div_cancel(b: Real, c: Real) {
    c != Real.0
    implies
    c * (b / c) = b
}

/// Multiplication with inverse commutes: a * (b * c) = b * (a * c).
theorem mul_inverse_comm(a: Real, b: Real, c: Real) {
    a * (b * c) = b * (a * c)
}

/// Division by product: a / (b * c) = (a / b) / c when b, c != 0.
/// This follows from inverse_dist and associativity.
// theorem div_mul_assoc(a: Real, b: Real, c: Real) {
//     b != Real.0 and c != Real.0
//     implies
//     a / (b * c) = (a / b) / c
// }

/// Cancellation: a / (b * a) = 1 / b when a, b != 0.
theorem div_mul_cancel_right(a: Real, b: Real) {
    a != Real.0 and b != Real.0
    implies
    a / (b * a) = Real.1 / b
}

/// Cancellation (left): (a * b) / a = b when a != 0.
theorem div_mul_cancel_left(a: Real, b: Real) {
    a != Real.0
    implies
    (a * b) / a = b
}

/// Bilateral cancellation: (a * b) / (c * b) = a / c when b, c != 0.
theorem div_cancel_common(a: Real, b: Real, c: Real) {
    b != Real.0 and c != Real.0
    implies
    (a * b) / (c * b) = a / c
}

/// Convert product equality to division equality.
/// If a * b = c * d, then a / d = c / b (when b, d != 0).
theorem prod_eq_to_div_eq(a: Real, b: Real, c: Real, d: Real) {
    a * b = c * d and b != Real.0 and d != Real.0
    implies
    a / d = c / b
}

/// Reciprocal of a division: (a/b).inverse = b/a when a, b != 0.
theorem inverse_div(a: Real, b: Real) {
    a != Real.0 and b != Real.0
    implies
    (a / b).inverse = b / a
}

/// Multiplication of fractions: (a/b) * (c/d) = (a*c) / (b*d) when b, d != 0.
theorem mul_div(a: Real, b: Real, c: Real, d: Real) {
    b != Real.0 and d != Real.0
    implies
    (a / b) * (c / d) = (a * c) / (b * d)
}

/// Division of fractions: (a/b) / (c/d) = (a*d) / (b*c) when b, c, d != 0.
theorem div_div(a: Real, b: Real, c: Real, d: Real) {
    b != Real.0 and c != Real.0 and d != Real.0
    implies
    (a / b) / (c / d) = (a * d) / (b * c)
}

/// Specific cancellation pattern: [(a*b)/(c*d)] / [b/d] = a/c when b, c, d != 0.
/// This is a direct consequence of div_div and simplification.
/// Currently commented out due to timeout on rearrangement steps.
// theorem div_cancel_pattern(a: Real, b: Real, c: Real, d: Real) {
//     b != Real.0 and c != Real.0 and d != Real.0
//     implies
//     ((a * b) / (c * d)) / (b / d) = a / c
// }

theorem geom_series(r: Real) {
    r.abs < 1 implies
    limit(partial(r.pow)) = 1 / (1 - r)
}

/// A geometric series with ratio of absolute value below one converges.
theorem geom_converges(r: Real) {
    r.abs < 1 implies converges(partial(r.pow))
}

/// Scaling a convergent sequence by a constant keeps it convergent.
theorem converges_mul_seq(a: Real, b: Nat -> Real) {
    converges(b) implies converges(mul_seq(a, b))
}

/// A convergent sequence converges to its limit.
theorem convergent_converges_to_limit(q: Nat -> Real) {
    converges(q) implies converges_to(q, limit(q))
}

/// A constant sequence converges.
theorem const_converges(a: Real) {
    converges(constant[Nat, Real](a))
}

/// A constant sequence converges to that constant.
theorem const_converges_to(a: Real) {
    converges_to(constant[Nat, Real](a), a)
}

/// The limit of a constant sequence is that constant.
theorem const_limit(a: Real) {
    limit(constant[Nat, Real](a)) = a
}

/// The limit of a sum of convergent sequences is the sum of the limits.
theorem limit_add_seq(a: Nat -> Real, b: Nat -> Real) {
    converges(a) and converges(b)
        implies converges_to(add_seq(a, b), limit(a) + limit(b))
}

/// A termwise comparison of convergent sequences is inherited by their limits.
theorem seq_lte_preserves_limit(a: Nat -> Real, b: Nat -> Real) {
    seq_lte(a, b) and converges(a) and converges(b) implies limit(a) <= limit(b)
}

/// A nonnegative series dominated termwise by a convergent series converges.
///
/// The comparison test.
theorem comparison_test(a: Nat -> Real, b: Nat -> Real) {
    is_lower_bound(a, Real.0) and seq_lte(a, b) and converges(partial(b))
        implies converges(partial(a))
}

// rectangular_sum.ac
numerals Nat

/// Convert a natural number to a real number.
define nat_to_real(n: Nat) -> Real {
    match n {
        Nat.zero {
            Real.0
        }
        Nat.suc(pred) {
            nat_to_real(pred) + Real.1
        }
    }
}

/// Constant two-argument function.
define const_fn_2(c: Real, i: Nat, j: Nat) -> Real {
    c
}

/// Helper function to compute the sum over row i.
define row_sum(f: (Nat, Nat) -> Real, m: Nat, i: Nat) -> Real {
    partial(f(i), m)
}

/// The sum of f over an m x n rectangle.
define rectangular_sum(f: (Nat, Nat) -> Real, m: Nat, n: Nat) -> Real {
    partial(row_sum(f, n), m)
}

/// The sum over an empty rectangle with no rows is zero.
theorem rectangular_sum_zero_rows(f: (Nat, Nat) -> Real, n: Nat) {
    rectangular_sum(f, Nat.0, n) = Real.0
}

/// The sum over an empty rectangle with no columns is zero.
theorem rectangular_sum_zero_cols(f: (Nat, Nat) -> Real, m: Nat) {
    rectangular_sum(f, m, Nat.0) = Real.0
}

/// Rectangular sum of a constant equals the product of the area and the constant.
theorem rectangular_sum_const(c: Real, m: Nat, n: Nat) {
    rectangular_sum(const_fn_2(c), m, n) = nat_to_real(m) * nat_to_real(n) * c
}

/// Rectangular sum of ones counts the number of lattice points.
theorem rectangular_sum_const_one(m: Nat, n: Nat) {
    rectangular_sum(const_fn_2(Real.1), m, n) = nat_to_real(m) * nat_to_real(n)
}

// cauchy_schwarz.ac
/// The pointwise product of two real sequences.
define pointwise_product(a: Nat -> Real, b: Nat -> Real, i: Nat) -> Real {
    a(i) * b(i)
}

/// The pointwise square of a real sequence.
define pointwise_square(a: Nat -> Real, i: Nat) -> Real {
    a(i) * a(i)
}

/// The finite dot product of two real sequences over the first `n` terms.
define finite_dot(a: Nat -> Real, b: Nat -> Real, n: Nat) -> Real {
    partial(pointwise_product(a, b), n)
}

/// The finite sum of squares of a real sequence over the first `n` terms.
define finite_square_sum(a: Nat -> Real, n: Nat) -> Real {
    partial(pointwise_square(a), n)
}

/// The square of a sum, with the two cross terms written separately.
theorem real_square_add_expanded(x: Real, y: Real) {
    (x + y) * (x + y) = x * x + x * y + (x * y + y * y)
}

/// Products of two products can be regrouped by their left and right factors.
theorem product_pair_square_rearrange(x: Real, y: Real) {
    (x * y) * (x * y) = (x * x) * (y * y)
}

/// The finite real Cauchy-Schwarz inequality in squared form.
theorem finite_cauchy_schwarz(a: Nat -> Real, b: Nat -> Real, n: Nat) {
    finite_dot(a, b, n) * finite_dot(a, b, n) <= finite_square_sum(a, n) * finite_square_sum(b, n)
}

// Real.exp.ac
/// The nth term in the Taylor series expansion of e^x.
/// For a real number x and natural number n, this returns x^n / n!
define exp_term(x: Real, n: Nat) -> Real {
    x.pow(n) / Real.from_rat(Rat.from_nat(n.factorial))
}

/// n.suc converted to Real is positive.
theorem suc_pos(n: Nat) {
    Real.from_rat(Rat.from_nat(n.suc)) > Real.0
}

let two = Real.1 + Real.1

/// Two is not zero.
theorem two_nonzero {
    two != Real.0
}

/// The exponential function x.exp = e^x.
/// Defined as the limit of the partial sums of the Taylor series.
attributes Real {
    /// The exponential function x.exp = e^x.
    /// Defined as the limit of the partial sums of the Taylor series.
    define exp(self) -> Real {
        limit(partial(exp_term(self)))
    }
}

/// Micro-lemma: a * (1 / b) = a / b
theorem mul_one_over(a: Real, b: Real) {
    b != Real.0 implies a * (Real.1 / b) = a / b
}

// harmonic.ac
/// The denominator of a harmonic-series term is positive.
theorem from_nat_suc_pos_real(n: Nat) {
    from_nat[Real](n.suc) > Real.0
}

/// The kth harmonic-series term, indexed from zero.
define harmonic(k: Nat) -> Real {
    Real.1 / from_nat[Real](k.suc)
}

/// Harmonic-series terms are positive.
theorem harmonic_pos(k: Nat) {
    harmonic(k) > Real.0
}

/// Harmonic-series terms are nonnegative.
theorem harmonic_nonnegative(k: Nat) {
    harmonic(k) >= Real.0
}

/// Each harmonic term in the dyadic block `[n, 2n)` is bounded below by `1 / (2n)`.
theorem harmonic_block_term_lower(n: Nat, k: Nat) {
    n > Nat.0 and k < n implies harmonic(n + k) >= Real.1 / from_nat[Real](n + n)
}


// derivative_linear.ac
/// A scalar factor in the numerator may be pulled outside a quotient.
theorem mul_div_left(c: Real, a: Real, b: Real) {
    (c * a) / b = c * (a / b)
}

// supremum.ac
// This file defines supremum-related concepts for sets of real numbers.

/// True if the set s is nonempty.
define is_nonempty(s: Set[Real]) -> Bool {
    exists(x: Real) {
        s.contains(x)
    }
}

/// True if b is an upper bound for the set s.
define is_set_upper_bound(s: Set[Real], b: Real) -> Bool {
    forall(x: Real) {
        s.contains(x) implies x <= b
    }
}

/// True if the set s has an upper bound.
define has_upper_bound(s: Set[Real]) -> Bool {
    exists(b: Real) {
        is_set_upper_bound(s, b)
    }
}

/// True if sup is the supremum (least upper bound) of the set s.
define is_set_supremum(s: Set[Real], sup: Real) -> Bool {
    is_set_upper_bound(s, sup) and
    forall(b: Real) {
        is_set_upper_bound(s, b) implies sup <= b
    }
}

/// Every nonempty set of real numbers that is bounded above has a supremum.
/// This is the completeness property of the real numbers.
theorem completeness(s: Set[Real]) {
    is_nonempty(s) and has_upper_bound(s)
    implies
    exists(sup: Real) {
        is_set_supremum(s, sup)
    }
}

/// If S ⊆ T, then sup S ≤ sup T.
theorem supremum_subset_le(s: Set[Real], t: Set[Real], sup_s: Real, sup_t: Real) {
    s.subset(t) and is_set_supremum(s, sup_s) and is_set_supremum(t, sup_t)
    implies
    sup_s <= sup_t
}

// log.ac
/// True if `y` is a logarithm of `x` with respect to the real exponential.
///
/// This predicate is declared here only because the `Real.log` method body
/// below resolves `choose_or_default(is_log(self), Real.0)` against the
/// interface. It is a method-body dependency, not part of the public API;
/// external packages never import it.
define is_log(x: Real, y: Real) -> Bool {
    y.exp = x
}

/// The real logarithm, defined exactly on positive reals.
attributes Real {
    /// The real logarithm, defined exactly on positive reals.
    define log(self) -> Option[Real] {
        if self > Real.0 {
            Option.some(choose_or_default(is_log(self), Real.0))
        } else {
            Option.none[Real]
        }
    }
}


/// Real exponentiation where it is defined over the reals.
/// Positive bases use `(exponent * base.log_or_zero).exp`; `0^0 = 1`; and `0^x = 0` for positive `x`.
attributes Real {
    /// Real exponentiation where it is defined over the reals.
    /// Positive bases use `(exponent * self.log_or_zero).exp`; `0^0 = 1`; and `0^x = 0` for positive `x`.
    define rpow(self, exponent: Real) -> Option[Real] {
        if self > Real.0 {
            Option.some((exponent * self.log.get_or_else(Real.0)).exp)
        } else {
            if self = Real.0 {
                if exponent = Real.0 {
                    Option.some(Real.1)
                } else {
                    if exponent > Real.0 {
                        Option.some(Real.0)
                    } else {
                        Option.none[Real]
                    }
                }
            } else {
                Option.none[Real]
            }
        }
    }
}

// Re-exports of the logarithm laws for external packages. Each declaration
// below matches a theorem proved in the corresponding implementation file
// (real/log.ac and real/finite_product_mean.ac).

/// A positive input has some logarithm value.
theorem log_some_of_pos_exists(x: Real) {
    x > Real.0 implies exists(y: Real) {
        x.log = Option.some(y)
    }
}

/// The defaulting logarithm witness is a right inverse to the exponential on positive reals.
theorem exp_log_or_zero(x: Real, y: Real) {
    x > Real.0 and x.log = Option.some(y) implies y.exp = x
}

/// The logarithm of one is zero.
theorem log_one {
    Real.1.log = Option.some(Real.0)
}

/// The logarithm of a positive product is the sum of the logarithms.
theorem log_mul(x: Real, y: Real, a: Real, b: Real) {
    x > Real.0 and y > Real.0 and x.log = Option.some(a) and y.log = Option.some(b)
    implies (x * y).log = Option.some(a + b)
}

/// Equal exponential values have equal arguments.
theorem exp_injective(x: Real, y: Real) {
    x.exp = y.exp implies x = y
}

/// The exponential of a natural multiple is the natural power of the exponential.
theorem exp_nat_mul(x: Real, n: Nat) {
    (from_nat[Real](n) * x).exp = x.exp.pow(n)
}

/// Natural-to-real counts are positive when the natural count is nonzero.
theorem from_nat_real_pos_of_ne_zero(n: Nat) {
    n != Nat.0 implies from_nat[Real](n) > Real.0
}

// log_inequalities.ac
/// The logarithm is monotone on positive reals.
theorem log_monotone(x: Real, y: Real, lx: Real, ly: Real) {
    x > Real.0 and y > Real.0 and x <= y and x.log = Option.some(lx) and y.log = Option.some(ly)
        implies lx <= ly
}

/// The logarithm is strictly monotone on positive reals.
theorem log_strict_monotone(x: Real, y: Real, lx: Real, ly: Real) {
    x > Real.0 and y > Real.0 and x < y and x.log = Option.some(lx) and y.log = Option.some(ly)
        implies lx < ly
}

// integral_exp.ac
/// Converting natural numbers to reals is nondecreasing.
theorem from_nat_lte_mono(m: Nat, n: Nat) {
    m <= n implies from_nat[Real](m) <= from_nat[Real](n)
}

// prod_seq.ac
/// This file defines the pointwise product of sequences and proves convergence properties.

/// The pointwise product of two sequences of real numbers.
define prod_seq(a: Nat -> Real, b: Nat -> Real, n: Nat) -> Real {
    a(n) * b(n)
}

// =============================================================================
// MAIN CONVERGENCE THEOREM
// =============================================================================

/// The product of two convergent sequences converges to the product of their limits.
theorem limit_prod_seq(a: Nat -> Real, b: Nat -> Real) {
    converges(a) and converges(b)
    implies
    converges_to(prod_seq(a, b), limit(a) * limit(b))
}

// sequence_limit_points.ac
/// A convergent sequence has a tail bound for every positive epsilon.
theorem converges_to_has_tail_bound(a: Nat -> Real, u: Real, eps: Real) {
    converges_to(a, u) and eps.is_positive implies exists(n: Nat) {
        tail_bound(a, u, n, eps)
    }
}

// sqrt.ac
/// The nonnegative square root, defined on nonnegative reals.
attributes Real {
    /// The nonnegative square root, defined on nonnegative reals.
    define sqrt(self) -> Option[Real] {
        self.rpow(Real.one_half)
    }
}

/// The square of the square root of a nonnegative real is the original number.
theorem sqrt_mul_self(x: Real) {
    x >= Real.0 implies exists(y: Real) {
        x.sqrt = Option.some(y) and y * y = x
    }
}

// sqrt_inequalities.ac
/// Squaring is order-reflecting on nonnegative reals.
theorem square_le_square_of_nonneg(a: Real, b: Real) {
    a >= Real.0 and b >= Real.0 and a * a <= b * b implies a <= b
}

/// A returned square-root value is nonnegative (sqrt is defined only on nonnegatives).
theorem sqrt_value_nonneg(x: Real, y: Real) {
    x.sqrt = Option.some(y) implies y >= Real.0
}

// Continuity and the intermediate value theorem, for top100 wrappers.

from order_set import closed_interval_set

/// True if f satisfies the delta-epsilon condition for continuity at x.
define continuous_condition(f: Real -> Real, x: Real, delta: Real, eps: Real) -> Bool {
    forall(x1: Real) {
        x1.is_close(x, delta) implies
        f(x1).is_close(f(x), eps)
    }
}

/// True if f is continuous at the point x.
define continuous_at(f: Real -> Real, x: Real) -> Bool {
    forall(eps: Real) {
        eps.is_positive implies exists(delta: Real) {
            delta.is_positive and continuous_condition(f, x, delta, eps)
        }
    }
}

/// True if f is continuous everywhere on the reals.
define continuous(f: Real -> Real) -> Bool {
    forall(x: Real) {
        continuous_at(f, x)
    }
}

/// A continuous real function on a closed interval takes every intermediate value.
theorem intermediate_value_closed_interval(
    f: Real -> Real, lower: Real, upper: Real, target: Real
) {
    continuous(f) and lower <= upper and f(lower) <= target and target <= f(upper)
    implies exists(point: Real) {
        closed_interval_set(lower, upper).contains(point) and f(point) = target
    }
}

// Re-exports of the continuity combinators for external packages.  Each
// declaration below matches a theorem or definition proved in the
// corresponding implementation file (real/continuity_composition.ac,
// real/continuity_sequences.ac, real/continuity_pointwise_mul.ac,
// real/continuity_const_add.ac, real/continuity_const_mul.ac).

/// The function obtained by adding the fixed real constant c to f on the left.
define const_add_left(c: Real, f: Real -> Real, x: Real) -> Real {
    c + f(x)
}

/// The function obtained by multiplying f on the left by the fixed real constant c.
define const_mul_left(c: Real, f: Real -> Real, x: Real) -> Real {
    c * f(x)
}

/// A constant real function is continuous.
theorem constant_function_is_continuous(c: Real) {
    continuous(constant[Real, Real](c))
}

/// The identity function on the reals is continuous.
theorem identity_function_is_continuous {
    continuous(identity_fn[Real])
}

/// Pointwise multiplication preserves continuous real functions.
theorem continuous_pointwise_mul(f: Real -> Real, g: Real -> Real) {
    continuous(f) and continuous(g) implies continuous(pointwise_mul[Real, Real](f, g))
}

/// Adding a fixed real constant on the left preserves continuous functions.
theorem continuous_const_add_left(c: Real, f: Real -> Real) {
    continuous(f) implies continuous(const_add_left(c, f))
}

/// Multiplication by a fixed real constant on the left preserves continuous functions.
theorem continuous_const_mul_left(c: Real, f: Real -> Real) {
    continuous(f) implies continuous(const_mul_left(c, f))
}

// Re-exports from the real submodules for external packages.
// Each declaration below matches a theorem or definition proved in the
// corresponding implementation file (real/abs_conv.ac, real/cauchy.ac,
// real/trig.ac, real/supremum.ac, real/pi.ac, real/limits.ac).

// real/abs_conv.ac
/// The absolute value of each element in a sequence.
define abs_fn(a: Nat -> Real) -> (Nat -> Real) {
    function(n: Nat) { a(n).abs }
}

/// True if the series of absolute values converges.
define absolutely_converges(a: Nat -> Real) -> Bool {
    converges(partial(abs_fn(a)))
}

/// An absolutely convergent series is convergent.
theorem absolutely_converges_imp_converges(a: Nat -> Real) {
    absolutely_converges(a) implies converges(partial(a))
}

// real/cauchy.ac
/// The coefficient function for the Cauchy product at index n.
define cauchy_coefficient(a: Nat -> Real, b: Nat -> Real, n: Nat) -> (Nat -> Real) {
    function(k: Nat) { a(k) * b(n - k) }
}

/// The Cauchy product of two sequences at index n.
define cauchy_product(a: Nat -> Real, b: Nat -> Real, n: Nat) -> Real {
    sum(map(n.suc.range, cauchy_coefficient(a, b, n)))
}

/// The sequence of Cauchy products.
define cauchy_seq(a: Nat -> Real, b: Nat -> Real) -> (Nat -> Real) {
    function(n: Nat) { cauchy_product(a, b, n) }
}

/// The Cauchy product of absolutely convergent series converges to the product
/// of their sums.
theorem cauchy_product_converges(a: Nat -> Real, b: Nat -> Real) {
    absolutely_converges(a) and absolutely_converges(b)
    implies
    converges_to(partial(cauchy_seq(a, b)), limit(partial(a)) * limit(partial(b)))
}

// real/trig.ac
/// The nth term in the Taylor series of sine: (-1)^n x^(2n+1) / (2n+1)!.
define sin_term(x: Real, n: Nat) -> Real {
    alternating_sign[Real](n) * x.pow(Nat.2 * n + Nat.1) / Real.from_rat(Rat.from_nat((Nat.2 * n + Nat.1).factorial))
}

/// The nth term in the Taylor series of cosine: (-1)^n x^(2n) / (2n)!.
define cos_term(x: Real, n: Nat) -> Real {
    alternating_sign[Real](n) * x.pow(Nat.2 * n) / Real.from_rat(Rat.from_nat((Nat.2 * n).factorial))
}

/// The sine and cosine functions, defined by their power series.
attributes Real {
    /// The sine function, defined by its power series.
    define sin(self) -> Real {
        limit(partial(sin_term(self)))
    }

    /// The cosine function, defined by its power series.
    define cos(self) -> Real {
        limit(partial(cos_term(self)))
    }
}

/// The sine series converges absolutely for every real number.
theorem sin_term_abs_converges(x: Real) {
    absolutely_converges(sin_term(x))
}

/// The cosine series converges absolutely for every real number.
theorem cos_term_abs_converges(x: Real) {
    absolutely_converges(cos_term(x))
}

/// Doubling a successor is two more than doubling.
theorem two_mul_suc(k: Nat) {
    Nat.2 * k.suc = (Nat.2 * k).suc.suc
}

// real/supremum.ac
/// A lower bound of a set of reals.
define is_set_lower_bound(s: Set[Real], b: Real) -> Bool {
    forall(x: Real) {
        s.contains(x) implies b <= x
    }
}

/// An infimum of a set of reals: a lower bound that is at least every lower bound.
define is_set_infimum(s: Set[Real], inf: Real) -> Bool {
    is_set_lower_bound(s, inf) and
    forall(b: Real) {
        is_set_lower_bound(s, b) implies b <= inf
    }
}

// real/pi.ac
/// Membership in the set of positive zeros of cosine in (0, 2].
define cos_zero_contains(x: Real) -> Bool {
    Real.0 < x and x <= two and x.cos = Real.0
}

/// The set of positive zeros of cosine in (0, 2].
let cos_zero_set = Set[Real].new(cos_zero_contains)

/// Half of pi: the infimum of the positive zeros of cosine in (0, 2].
let pi_over_two: Real satisfy {
    is_set_infimum(cos_zero_set, pi_over_two)
}

/// The mathematical constant pi.
let pi = two * pi_over_two

/// The cosine of pi is negative one.
theorem cos_pi_neg_one {
    pi.cos = -Real.1
}

/// The sine of pi is zero.
theorem sin_pi_zero {
    pi.sin = Real.0
}

/// Pi is positive.
theorem pi_pos {
    pi > Real.0
}

/// Pi is less than four.
theorem pi_lt_four {
    pi < two * two
}

/// Two times two is four.
theorem two_mul_two_eq_four {
    two * two = Real.from_rat(Rat.from_nat(Nat.4))
}

/// Natural real exponents agree with ordinary natural powers for positive bases.
theorem rpow_nat(base: Real, n: Nat) {
    base > Real.0 implies base.rpow(from_nat[Real](n)) = Option.some(base.pow(n))
}

// real/limits.ac
/// The subsequence of `a` selected by an index map.
define subsequence(a: Nat -> Real, f: Nat -> Nat, n: Nat) -> Real {
    compose(a, f)(n)
}

/// If a sequence converges, then its even-indexed subsequence converges to the same limit.
theorem converges_subsequence_double(a: Nat -> Real) {
    converges(a)
    implies
    converges_to(subsequence(a, Nat.2.mul), limit(a))
}


/// True if an index map is unbounded, i.e. not bounded above by any fixed
/// natural number.
define is_unbounded(f: Nat -> Nat) -> Bool {
    forall(bound: Nat) {
        exists(n: Nat) {
            bound < f(n)
        }
    }
}

// real/bolzano_weierstrass.ac
/// The Bolzano-Weierstrass theorem: every bounded sequence of real numbers
/// has a convergent subsequence, encoded by a nondecreasing unbounded index
/// map `f` and a limit `l`.
theorem bolzano_weierstrass(a: Nat -> Real) {
    exists(lo: Real, hi: Real) {
        forall(n: Nat) { lo <= a(n) and a(n) <= hi }
    } implies
        exists(f: Nat -> Nat, l: Real) {
            is_monotone(f) and is_unbounded(f) and converges_to(subsequence(a, f), l)
        }
}

// real/Real.exp.ac
/// The inverse relation from the binomial coefficient identity:
/// 1 / (k! * (n-k)!) = C(n,k) / n!.
theorem choose_factorial_inverse(n: Nat, k: Nat) {
    k <= n implies Real.1 / (Real.from_rat(Rat.from_nat(k.factorial)) * Real.from_rat(Rat.from_nat((n - k).factorial))) = from_nat[Real](n.binom(k)) / Real.from_rat(Rat.from_nat(n.factorial))
}

// Re-exports of the exponential-function laws for external packages. Each
// declaration matches a theorem proved in real/Real.exp.ac.

/// The exponential addition law: (x + y).exp = x.exp * y.exp.
theorem exp_add(x: Real, y: Real) {
    (x + y).exp = x.exp * y.exp
}

/// The exponential of zero is one.
theorem exp_zero {
    (Real.0).exp = Real.1
}

/// The zeroth term in the exponential series is always 1.
theorem exp_term_zero_index(x: Real) {
    exp_term(x, Nat.0) = Real.1
}

/// The first term in the exponential series is x.
theorem exp_term_one_index(x: Real) {
    exp_term(x, Nat.1) = x
}

/// The partial sums of the exponential series converge for every real number.
theorem exp_term_partial_converges(x: Real) {
    converges(partial(exp_term(x)))
}

/// The multiplication form of the exp_term recurrence:
/// exp_term(x, n+1) * (n+1) = x * exp_term(x, n).
theorem exp_term_mul_recurrence(x: Real, n: Nat) {
    exp_term(x, n.suc) * Real.from_rat(Rat.from_nat(n.suc)) = x * exp_term(x, n)
}

/// The exponential function is always positive.
theorem exp_pos(x: Real) {
    x.exp > Real.0
}

/// For positive x, every exponential-series term is positive.
theorem exp_term_pos(x: Real, n: Nat) {
    x > Real.0 implies exp_term(x, n) > Real.0
}

// Re-exports of the sequence tail and scaled-limit laws for external
// packages. Each declaration matches a theorem or definition proved in
// real/real_series.ac.

/// The tail of a sequence: the sequence `a` shifted to start at index `n`.
define tail[T](a: Nat -> T, n: Nat, i: Nat) -> T {
    a(n + i)
}

/// A convergent tail makes the whole sequence converge to the tail's limit.
theorem tail_imp_converges_to(a: Nat -> Real, n: Nat) {
    converges(tail(a, n)) implies
    converges_to(a, limit(tail(a, n)))
}

/// Scaling a convergent sequence preserves convergence, to the scaled limit.
theorem mul_seq_converges_to(a: Real, b: Nat -> Real) {
    converges(b) implies converges_to(mul_seq(a, b), a * limit(b))
}

// Re-export of a sequence fact for external packages. The declaration
// matches a theorem proved in real/lhopital.ac.

/// Pointwise equal sequences converge to the same limit.
theorem seq_pointwise_eq_converges_to(u: Nat -> Real, v: Nat -> Real, x: Real) {
    (forall(n: Nat) { u(n) = v(n) }) and converges_to(u, x)
    implies converges_to(v, x)
}

// ---------------------------------------------------------------------------
// The Riemann (Darboux) integral API (real/integral.ac, real/integral_exp.ac,
// real/integral_trig.ac, real/supremum.ac)
// ---------------------------------------------------------------------------
//
// The declarations below mirror theorems and definitions proved in the
// implementation modules of the real package; the package validation checks
// that each interface declaration matches exactly one implementation
// declaration with the same statement, so every name exposed here is backed
// by a proof in the library rather than by an axiom.

/// The quotient (f(x) - f(x0)) / (x - x0).
define difference_quotient(f: Real -> Real, x0: Real, x: Real) -> Real {
    (f(x) - f(x0)) / (x - x0)
}

/// True if a real-valued function has derivative d at a point.
define has_derivative_at(f: Real -> Real, x0: Real, d: Real) -> Bool {
    forall(eps: Real) {
        eps.is_positive implies exists(delta: Real) {
            delta.is_positive and forall(x: Real) {
                x != x0 and x.is_close(x0, delta)
                implies difference_quotient(f, x0, x).is_close(d, eps)
            }
        }
    }
}

/// A function df is the pointwise derivative of f on all real inputs.
define is_derivative_fn(f: Real -> Real, df: Real -> Real) -> Bool {
    forall(x: Real) {
        has_derivative_at(f, x, df(x))
    }
}

// real/darboux.ac
/// Darboux's theorem: derivatives have the intermediate value property.  If
/// `f` is differentiable everywhere and `df(a) < t < df(b)`, then some `c`
/// strictly between `a` and `b` has `df(c) = t`.
theorem darboux_theorem(f: Real -> Real, df: Real -> Real, a: Real, b: Real, t: Real) {
    a < b and
    (forall(x: Real) { has_derivative_at(f, x, df(x)) }) and
    df(a) < t and t < df(b)
    implies exists(c: Real) {
        a < c and c < b and df(c) = t
    }
}

/// True if x lies in the closed interval [a, b].
define interval_contains(a: Real, b: Real, x: Real) -> Bool {
    a <= x and x <= b
}

/// The closed interval [a, b] as a set of reals.
define interval_set(a: Real, b: Real) -> Set[Real] {
    Set[Real].new(interval_contains(a, b))
}

/// The image of a set s under function f.
define function_image(f: Real -> Real, s: Set[Real]) -> Set[Real] {
    set_image(s, f)
}

/// The image of the interval [x, y] under f.
define interval_image(f: Real -> Real, x: Real, y: Real) -> Set[Real] {
    function_image(f, interval_set(x, y))
}

/// True if s has a lower bound.
define has_lower_bound(s: Set[Real]) -> Bool {
    exists(b: Real) {
        is_set_lower_bound(s, b)
    }
}

/// True if p is nondecreasing up to index n.
define partition_monotone(p: Nat -> Real, n: Nat) -> Bool {
    forall(i: Nat, j: Nat) {
        i <= j and j <= n implies p(i) <= p(j)
    }
}

/// True if p is a partition of [a, b] of length n:
/// p(0) = a, p(n) = b, and p is nondecreasing.
define is_partition(p: Nat -> Real, a: Real, b: Real, n: Nat) -> Bool {
    (p(Nat.0) = a) and (p(n) = b) and partition_monotone(p, n)
}

/// The successive-difference sequence of p.
define diff_step(p: Nat -> Real, i: Nat) -> Real {
    p(i + 1) - p(i)
}

/// The supremum of f on the interval [x, y], with fallback value zero.
let interval_sup(f: Real -> Real, x: Real, y: Real) -> m: Real satisfy {
    if is_nonempty(interval_set(x, y)) and has_upper_bound(interval_image(f, x, y)) {
        is_set_supremum(interval_image(f, x, y), m)
    } else {
        m = Real.0
    }
}

/// The infimum of f on the interval [x, y], with fallback value zero.
let interval_inf(f: Real -> Real, x: Real, y: Real) -> m: Real satisfy {
    if is_nonempty(interval_set(x, y)) and has_lower_bound(interval_image(f, x, y)) {
        is_set_infimum(interval_image(f, x, y), m)
    } else {
        m = Real.0
    }
}

/// The lower step of a partition at index i.
define partition_step_lower(f: Real -> Real, p: Nat -> Real, i: Nat) -> Real {
    interval_inf(f, p(i), p(i + 1)) * diff_step(p, i)
}

/// The upper step of a partition at index i.
define partition_step_upper(f: Real -> Real, p: Nat -> Real, i: Nat) -> Real {
    interval_sup(f, p(i), p(i + 1)) * diff_step(p, i)
}

/// The lower Darboux sum of f over the partition p of length n.
define lower_sum(f: Real -> Real, p: Nat -> Real, n: Nat) -> Real {
    partial(partition_step_lower(f, p), n)
}

/// The upper Darboux sum of f over the partition p of length n.
define upper_sum(f: Real -> Real, p: Nat -> Real, n: Nat) -> Real {
    partial(partition_step_upper(f, p), n)
}

/// True if s is the lower Darboux sum of f over some partition of [a, b].
define lower_sum_contains(f: Real -> Real, a: Real, b: Real, s: Real) -> Bool {
    exists(p: Nat -> Real, n: Nat) {
        is_partition(p, a, b, n) and s = lower_sum(f, p, n)
    }
}

/// The set of lower Darboux sums of f over all partitions of [a, b].
define lower_sum_set(f: Real -> Real, a: Real, b: Real) -> Set[Real] {
    Set[Real].new(lower_sum_contains(f, a, b))
}

/// True if s is the upper Darboux sum of f over some partition of [a, b].
define upper_sum_contains(f: Real -> Real, a: Real, b: Real, s: Real) -> Bool {
    exists(p: Nat -> Real, n: Nat) {
        is_partition(p, a, b, n) and s = upper_sum(f, p, n)
    }
}

/// The set of upper Darboux sums of f over all partitions of [a, b].
define upper_sum_set(f: Real -> Real, a: Real, b: Real) -> Set[Real] {
    Set[Real].new(upper_sum_contains(f, a, b))
}

/// True if f is integrable on [a, b]: the supremum of its lower sums equals
/// the infimum of its upper sums.
define is_integrable(f: Real -> Real, a: Real, b: Real) -> Bool {
    exists(m: Real) {
        is_set_supremum(lower_sum_set(f, a, b), m) and is_set_infimum(upper_sum_set(f, a, b), m)
    }
}

/// The Riemann integral of f over [a, b]; zero when f is not integrable.
let integral(f: Real -> Real, a: Real, b: Real) -> v: Real satisfy {
    if is_integrable(f, a, b) {
        is_set_supremum(lower_sum_set(f, a, b), v) and is_set_infimum(upper_sum_set(f, a, b), v)
    } else {
        v = Real.0
    }
}

/// Negating an inequality reverses it.
theorem neg_lte_flip(a: Real, b: Real) {
    a <= b implies -b <= -a
}

/// The fundamental theorem of calculus, part 2: if g is continuous with
/// pointwise derivative f, f is bounded on [a, b] and integrable there, then
/// the integral of f over [a, b] is g(b) - g(a).
theorem ftc2_general(f: Real -> Real, g: Real -> Real, a: Real, b: Real, lb: Real, ub: Real) {
    a <= b and continuous(g) and is_derivative_fn(g, f) and is_integrable(f, a, b) and
    (forall(t: Real) { interval_contains(a, b, t) implies lb <= f(t) }) and
    (forall(t: Real) { interval_contains(a, b, t) implies f(t) <= ub })
    implies
    integral(f, a, b) = g(b) - g(a)
}

/// An m-Lipschitz function on [a, b] bounded there, with known continuous
/// antiderivative, is integrable on [a, b].
theorem fn_integrable_gen(f: Real -> Real, g: Real -> Real, a: Real, b: Real, m: Real, lb: Real, ub: Real) {
    a <= b and Real.0 <= m and continuous(g) and is_derivative_fn(g, f) and
    (forall(u: Real, v: Real) {
        interval_contains(a, b, u) and interval_contains(a, b, v) implies (f(u) - f(v)).abs <= m * (u - v).abs
    }) and
    (forall(t: Real) { interval_contains(a, b, t) implies lb <= f(t) }) and
    (forall(t: Real) { interval_contains(a, b, t) implies f(t) <= ub }) implies
    is_integrable(f, a, b)
}

/// Constant real functions have the constant-zero function as global derivative.
theorem derivative_fn_constant(c: Real) {
    is_derivative_fn(constant[Real, Real](c), constant[Real, Real](Real.0))
}

/// The identity function has the constant-one function as global derivative.
theorem derivative_fn_identity {
    is_derivative_fn(identity_fn[Real], constant[Real, Real](Real.1))
}

/// Pointwise negation transports global derivative functions.
theorem derivative_fn_neg(f: Real -> Real, df: Real -> Real) {
    is_derivative_fn(f, df) implies is_derivative_fn(pointwise_neg(f), pointwise_neg(df))
}

/// Pointwise addition transports global derivative functions.
theorem derivative_fn_add(f: Real -> Real, g: Real -> Real, df: Real -> Real, dg: Real -> Real) {
    is_derivative_fn(f, df) and is_derivative_fn(g, dg)
        implies is_derivative_fn(pointwise_add(f, g), pointwise_add(df, dg))
}

/// Pointwise squares transport global derivative functions via the product rule.
theorem derivative_fn_square(f: Real -> Real, df: Real -> Real) {
    is_derivative_fn(f, df)
        implies is_derivative_fn(pointwise_mul(f, f), pointwise_add(pointwise_mul(f, df), pointwise_mul(f, df)))
}

/// Left scalar multiplication transports global derivative functions.
theorem derivative_fn_const_mul(c: Real, f: Real -> Real, df: Real -> Real) {
    is_derivative_fn(f, df)
        implies is_derivative_fn(pointwise_mul(constant[Real, Real](c), f), pointwise_mul(constant[Real, Real](c), df))
}

/// The sine function is everywhere differentiable with derivative cosine.
theorem sin_is_derivative_fn {
    is_derivative_fn(Real.sin, Real.cos)
}

/// The cosine function is everywhere differentiable with derivative negative sine.
theorem cos_derivative_is_neg_sin {
    is_derivative_fn(Real.cos, pointwise_neg(Real.sin))
}

/// Pointwise negation preserves continuous real functions.
theorem continuous_pointwise_neg(f: Real -> Real) {
    continuous(f) implies continuous(pointwise_neg(f))
}

/// Pointwise addition preserves continuous real functions.
theorem continuous_pointwise_add(f: Real -> Real, g: Real -> Real) {
    continuous(f) and continuous(g) implies continuous(pointwise_add(f, g))
}

/// The sine of zero is zero.
theorem sin_zero {
    (Real.0).sin = Real.0
}

/// The cosine of zero is one.
theorem cos_zero {
    (Real.0).cos = Real.1
}

/// The sine of x plus y is (x).sincos(y) + x.cos(y).sin.
theorem sin_add(x: Real, y: Real) {
    (x + y).sin = x.sin * y.cos + x.cos * y.sin
}

/// The Pythagorean identity: x.sin^2 + x.cos^2 = 1.
theorem sin_sq_add_cos_sq(x: Real) {
    x.sin.pow(Nat.2) + x.cos.pow(Nat.2) = Real.1
}

/// Sine is one-Lipschitz: |x.sin - y.sin| <= |x - y|.
theorem sin_lipschitz(x: Real, y: Real) {
    (x.sin - y.sin).abs <= (x - y).abs
}

/// Cosine is one-Lipschitz: |x.cos - y.cos| <= |x - y|.
theorem cos_lipschitz(x: Real, y: Real) {
    (x.cos - y.cos).abs <= (x - y).abs
}

/// The absolute value of sine is at most one.
theorem sin_abs_le_one(x: Real) {
    x.sin.abs <= Real.1
}

/// The absolute value of cosine is at most one.
theorem cos_abs_le_one(x: Real) {
    x.cos.abs <= Real.1
}

/// The sine of two pi is zero.
theorem sin_two_pi_zero {
    (pi + pi).sin = Real.0
}

/// The cosine of two pi is one.
theorem cos_two_pi_one {
    (pi + pi).cos = Real.1
}

/// Sine is continuous.
theorem sin_continuous {
    continuous(Real.sin)
}

/// Cosine is continuous.
theorem cos_continuous {
    continuous(Real.cos)
}

/// The absolute value of a nonnegative real is itself.
theorem abs_of_nonneg(x: Real) {
    x >= Real.0 implies x.abs = x
}

/// Two is positive.
theorem two_positive {
    two.is_positive
}

// Re-exports needed by the Fourier foundations module
// (analysis/fourier_foundations.ac).  Each declaration matches a theorem
// proved in real/trig.ac, real/trig_identities.ac, or real/integral_trig.ac.

/// The cosine addition law: (x + y).cos = x.cos y.cos - x.sin y.sin.
theorem cos_add(x: Real, y: Real) {
    (x + y).cos = x.cos * y.cos - x.sin * y.sin
}

/// Cosine is even.
theorem cos_neg(x: Real) {
    (-x).cos = x.cos
}

/// Sine is odd.
theorem sin_neg(x: Real) {
    (-x).sin = -x.sin
}

/// The integral of sine over [a, b] is a.cos - b.cos.
theorem integral_sin(a: Real, b: Real) {
    a <= b implies integral(Real.sin, a, b) = a.cos - b.cos
}

/// The integral of cosine over [a, b] is b.sin - a.sin.
theorem integral_cos(a: Real, b: Real) {
    a <= b implies integral(Real.cos, a, b) = b.sin - a.sin
}

/// The integral of sine over [0, pi] is two.
theorem integral_sin_zero_pi {
    integral(Real.sin, Real.0, pi) = two
}

/// Pointwise products transport global derivative functions by the product rule.
theorem derivative_fn_mul(f: Real -> Real, g: Real -> Real, df: Real -> Real, dg: Real -> Real) {
    is_derivative_fn(f, df) and is_derivative_fn(g, dg)
        implies is_derivative_fn(pointwise_mul(f, g), pointwise_add(pointwise_mul(f, dg), pointwise_mul(g, df)))
}

/// Any member of an interval is above the left endpoint.
theorem interval_contains_left(a: Real, b: Real, x: Real) {
    interval_contains(a, b, x) implies a <= x
}

/// Any member of an interval is below the right endpoint.
theorem interval_contains_right(a: Real, b: Real, x: Real) {
    interval_contains(a, b, x) implies x <= b
}

/// The sequence n -> 1/(n+1) converges.
theorem one_over_suc_converges {
    converges(harmonic)
}

/// The limit of the sequence n -> 1/(n+1) is zero.
theorem limit_one_over_suc {
    limit(harmonic) = Real.0
}

// real/derivative_basic.ac, real/mean_value.ac
//
// The mean value theorem and its classical consequences (Rolle's theorem).
// The declarations below mirror definitions and theorems proved in the
// implementation modules `real/derivative_basic.ac` and `real/mean_value.ac`;
// the package validation checks that each interface declaration matches
// exactly one implementation declaration with the same statement.

/// True if f is differentiable at the point x0.
define differentiable_at(f: Real -> Real, x0: Real) -> Bool {
    exists(d: Real) {
        has_derivative_at(f, x0, d)
    }
}

/// True if f is differentiable at every point of the open interval (lower, upper).
define differentiable_on_open(f: Real -> Real, lower: Real, upper: Real) -> Bool {
    forall(x: Real) {
        lower < x and x < upper implies differentiable_at(f, x)
    }
}

/// True if f is continuous at every point of the closed interval [lower, upper].
define continuous_on_closed(f: Real -> Real, lower: Real, upper: Real) -> Bool {
    forall(x: Real) {
        closed_interval_set(lower, upper).contains(x) implies continuous_at(f, x)
    }
}

/// Rolle's theorem: a continuous function on a closed interval that agrees
/// at the endpoints and is differentiable in the interior has a stationary
/// point (a zero of the derivative) in the interior.
theorem rolle_theorem(f: Real -> Real, a: Real, b: Real) {
    continuous_on_closed(f, a, b) and a < b and differentiable_on_open(f, a, b) and
    f(a) = f(b)
    implies exists(c: Real) {
        a < c and c < b and has_derivative_at(f, c, Real.0)
    }
}

/// The slope of the secant line through (a, f(a)) and (b, f(b)).
define secant_slope(f: Real -> Real, a: Real, b: Real) -> Real {
    (f(b) - f(a)) / (b - a)
}

/// The mean value theorem: a continuous function on a closed interval that
/// is differentiable everywhere has an interior point whose derivative
/// equals the slope of the secant through the two endpoint values.
theorem mean_value_theorem(f: Real -> Real, df: Real -> Real, a: Real, b: Real) {
    continuous(f) and a < b and is_derivative_fn(f, df)
    implies exists(c: Real) {
        a < c and c < b and has_derivative_at(f, c, secant_slope(f, a, b))
    }
}
