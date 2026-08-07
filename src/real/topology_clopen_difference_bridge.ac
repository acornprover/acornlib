/// Clopen real-set difference bridge.

from data.basic.set import Set, compl_contains_eq, difference_contains_eq, intersection_contains_eq,
    set_ext
from real.real_field import Real
from real.topology import is_closed_set, is_open_set
from real.topology_clopen import clopen_real_set_intro, clopen_real_set_is_closed,
    clopen_real_set_is_open, is_clopen_real_set
from real.topology_complements import complement_of_closed_is_open
from real.topology_difference import difference_of_closed_and_open_is_closed
from real.topology_open_intersection import intersection_of_open_is_open

/// Set difference is intersection with complement for real sets.
theorem real_difference_eq_intersection_complement(s: Set[Real], t: Set[Real]) {
    s.difference(t) = s.intersection(t.c)
} by {
    let l = s.difference(t)
    let r = s.intersection(t.c)
    forall(x: Real) {
        difference_contains_eq[Real](s, t, x)
        intersection_contains_eq[Real](s, t.c, x)
        compl_contains_eq[Real](t, x)
        if l.contains(x) {
            difference_contains_eq[Real](s, t, x)
            s.contains(x)
            not t.contains(x)
            compl_contains_eq[Real](t, x)
            t.c.contains(x)
            intersection_contains_eq[Real](s, t.c, x)
            r.contains(x)
        }
        if r.contains(x) {
            intersection_contains_eq[Real](s, t.c, x)
            s.contains(x)
            t.c.contains(x)
            compl_contains_eq[Real](t, x)
            not t.contains(x)
            difference_contains_eq[Real](s, t, x)
            l.contains(x)
        }
        l.contains(x) = r.contains(x)
    }
    set_ext[Real](l, r)
}

/// The difference of two clopen real sets is open.
theorem difference_of_clopen_real_sets_is_open(s: Set[Real], t: Set[Real]) {
    is_clopen_real_set(s) and is_clopen_real_set(t) implies is_open_set(s.difference(t))
} by {
    if is_clopen_real_set(s) and is_clopen_real_set(t) {
        clopen_real_set_is_open(s)
        clopen_real_set_is_closed(t)
        is_open_set(s)
        is_closed_set(t)
        complement_of_closed_is_open(t)
        is_open_set(t.c)
        intersection_of_open_is_open(s, t.c)
        is_open_set(s.intersection(t.c))
        real_difference_eq_intersection_complement(s, t)
        s.difference(t) = s.intersection(t.c)
        is_open_set(s.difference(t))
    }
}

/// The difference of two clopen real sets is closed.
theorem difference_of_clopen_real_sets_is_closed(s: Set[Real], t: Set[Real]) {
    is_clopen_real_set(s) and is_clopen_real_set(t) implies is_closed_set(s.difference(t))
} by {
    if is_clopen_real_set(s) and is_clopen_real_set(t) {
        clopen_real_set_is_closed(s)
        clopen_real_set_is_open(t)
        is_closed_set(s)
        is_open_set(t)
        difference_of_closed_and_open_is_closed(s, t)
        is_closed_set(s.difference(t))
    }
}

/// The difference of two clopen real sets is clopen.
theorem difference_of_clopen_real_sets_is_clopen(s: Set[Real], t: Set[Real]) {
    is_clopen_real_set(s) and is_clopen_real_set(t) implies
    is_clopen_real_set(s.difference(t))
} by {
    if is_clopen_real_set(s) and is_clopen_real_set(t) {
        difference_of_clopen_real_sets_is_open(s, t)
        difference_of_clopen_real_sets_is_closed(s, t)
        is_open_set(s.difference(t))
        is_closed_set(s.difference(t))
        clopen_real_set_intro(s.difference(t))
        is_clopen_real_set(s.difference(t))
    }
}
