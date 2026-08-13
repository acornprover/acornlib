/// The Bolzano–Weierstrass theorem: every bounded real sequence has a
/// convergent subsequence.
///
/// This file proves the Bolzano–Weierstrass theorem (Bernard Bolzano, 1817;
/// Karl Weierstrass, 1860s): every bounded sequence of real numbers has a
/// convergent subsequence.  The proof is the classical one through the
/// monotone-subsequence theorem: every real sequence has a nondecreasing or
/// nonincreasing subsequence (the "peaks" argument), and a monotone
/// subsequence of a bounded sequence converges by the monotone convergence
/// principle.
///
/// Call `m` a *peak* of the sequence `a` if `a(m)` is at least every later
/// value, `a(k) <= a(m)` for every `k > m`.  If peaks occur beyond every
/// index, the successive peaks form a nonincreasing subsequence.  Otherwise
/// no peaks occur beyond some index `m0`; then every index at or beyond
/// `m0 + 1` is followed by a strictly larger value, and repeatedly choosing
/// such a later index builds a strictly increasing subsequence.  Either way
/// the selected subsequence is monotone and, being a subsequence of a
/// bounded sequence, bounded; the monotone convergence principle
/// (`monotone_convergence_principle`) makes it converge.
///
/// The subsequence is encoded by an index map `f: Nat -> Nat` that is
/// nondecreasing and unbounded (the library's `is_subsequence_index`), with
/// the selected subsequence `subsequence(a, f) = compose(a, f)`.

from nat import Nat, lt_suc, lte_ref, lte_trans, lt_trans, lte_antisymm,
    lt_imp_lte_suc, lt_and_lte, lte_and_lt, lte_suc_suc, add_suc_right, add_suc_left,
    lte_add_left, lte_add_right, lt_or_lte
from order import is_monotone, monotone_from_forall, not_lte_imp_gt, lte_imp_not_lt,
    lt_imp_lte
from real.real_base import Real
from real.real_seq import converges, converges_to, limit, converges_imp_converges_to,
    convergent_converges_to_limit
from real.real_series import is_increasing, is_upper_bound, is_lower_bound,
    monotone_convergence_principle, is_decreasing_seq, neg_seq, decreasing_neg_increasing,
    neg_seq_converges_converse
from real.limits import is_subsequence_index, subsequence, is_nondecreasing_nat, is_unbounded,
    subsequence_index_is_monotone, subsequence_index_is_unbounded,
    subsequence_index_of_monotone_unbounded, subsequence_eq_compose
from real.mean_value import lte_imp_neg_lte_neg

numerals Real

/// True if `m` is a peak of the sequence `a`: no later term exceeds `a(m)`.
define is_peak(a: Nat -> Real, m: Nat) -> Bool {
    forall(k: Nat) {
        m < k implies a(k) <= a(m)
    }
}

/// A peak is at least every later value.
theorem peak_later_le(a: Nat -> Real, m: Nat, k: Nat) {
    is_peak(a, m) and m < k implies a(k) <= a(m)
} by {
    if is_peak(a, m) and m < k {
        is_peak(a, m) = forall(k0: Nat) {
            m < k0 implies a(k0) <= a(m)
        }
        m < k implies a(k) <= a(m)
        a(k) <= a(m)
    }
}

/// An index that is not a peak has a later index with a strictly larger value.
theorem not_peak_imp_later_strict(a: Nat -> Real, x: Nat) {
    (not is_peak(a, x)) implies exists(k: Nat) { x < k and a(x) < a(k) }
} by {
    if not is_peak(a, x) {
        is_peak(a, x) = forall(k: Nat) {
            x < k implies a(k) <= a(x)
        }
        not forall(k: Nat) {
            x < k implies a(k) <= a(x)
        }
        exists(k: Nat) {
            not (x < k implies a(k) <= a(x))
        }
        let k: Nat satisfy {
            not (x < k implies a(k) <= a(x))
        }
        x < k
        not (a(k) <= a(x))
        not_lte_imp_gt(a(k), a(x))
        a(k) > a(x)
        a(x) < a(k)
        exists(z: Nat) { x < z and a(x) < a(z) }
    }
}

/// Zero is at most every natural number.
theorem nat_zero_lte(n: Nat) {
    Nat.0 <= n
} by {
    define p(k: Nat) -> Bool {
        Nat.0 <= k
    }
    lte_ref(Nat.0)
    p(Nat.0)
    forall(k: Nat) {
        if p(k) {
            lt_suc(k)
            k < k.suc
            lt_imp_lte(k, k.suc)
            k <= k.suc
            lte_trans(Nat.0, k, k.suc)
            Nat.0 <= k.suc
            p(k.suc)
        }
    }
    p(Nat.0) and forall(k: Nat) { p(k) implies p(k.suc) }
    Nat.induction(p)
    p(n)
    Nat.0 <= n
}

/// A successor index after `x` at which the sequence has a peak, when such
/// an index exists.
let peak_after(a: Nat -> Real, x: Nat) -> y: Nat satisfy {
    (exists(z: Nat) { x < z and is_peak(a, z) }) implies (x < y and is_peak(a, y))
}

/// A successor index after `x` at which the sequence value strictly
/// increases, when such an index exists.
let strict_after(a: Nat -> Real, x: Nat) -> y: Nat satisfy {
    (exists(z: Nat) { x < z and a(x) < a(z) }) implies (x < y and a(x) < a(y))
}

/// The defining property of `peak_after`.
theorem peak_after_spec(a: Nat -> Real, x: Nat) {
    (exists(z: Nat) { x < z and is_peak(a, z) })
    implies (x < peak_after(a, x) and is_peak(a, peak_after(a, x)))
}

/// The defining property of `strict_after`.
theorem strict_after_spec(a: Nat -> Real, x: Nat) {
    (exists(z: Nat) { x < z and a(x) < a(z) })
    implies (x < strict_after(a, x) and a(x) < a(strict_after(a, x)))
}

/// The successive peaks of a sequence, one per step.
define peak_index(a: Nat -> Real, n: Nat) -> Nat {
    match n {
        Nat.zero { peak_after(a, Nat.0) }
        Nat.suc(pred) { peak_after(a, peak_index(a, pred)) }
    }
}

/// A strictly increasing index sequence starting at a given base index.
define strict_index(a: Nat -> Real, base: Nat, n: Nat) -> Nat {
    match n {
        Nat.zero { base }
        Nat.suc(pred) { strict_after(a, strict_index(a, base, pred)) }
    }
}

/// Under unbounded peaks, the peak-index map strictly increases at each
/// step.
theorem peak_index_strict_at(a: Nat -> Real, n: Nat) {
    (forall(m: Nat) { exists(p: Nat) { m < p and is_peak(a, p) } })
    implies peak_index(a, n) < peak_index(a, n.suc)
} by {
    if forall(m: Nat) { exists(p: Nat) { m < p and is_peak(a, p) } } {
        define p(k: Nat) -> Bool {
            peak_index(a, k) < peak_index(a, k.suc)
        }
        exists(pk: Nat) { Nat.0 < pk and is_peak(a, pk) }
        peak_after_spec(a, Nat.0)
        Nat.0 < peak_after(a, Nat.0) and is_peak(a, peak_after(a, Nat.0))
        exists(pk: Nat) { peak_after(a, Nat.0) < pk and is_peak(a, pk) }
        peak_after_spec(a, peak_after(a, Nat.0))
        peak_after(a, Nat.0) < peak_after(a, peak_after(a, Nat.0)) and
            is_peak(a, peak_after(a, peak_after(a, Nat.0)))
        peak_index(a, Nat.0) = peak_after(a, Nat.0)
        peak_index(a, Nat.0.suc) = peak_after(a, peak_index(a, Nat.0))
        peak_index(a, Nat.0) < peak_index(a, Nat.0.suc)
        p(Nat.0)
        forall(k: Nat) {
            if p(k) {
                exists(pk: Nat) { peak_index(a, k) < pk and is_peak(a, pk) }
                peak_after_spec(a, peak_index(a, k))
                peak_index(a, k) < peak_after(a, peak_index(a, k)) and
                    is_peak(a, peak_after(a, peak_index(a, k)))
                peak_index(a, k.suc) = peak_after(a, peak_index(a, k))
                exists(pk: Nat) { peak_index(a, k.suc) < pk and is_peak(a, pk) }
                peak_after_spec(a, peak_index(a, k.suc))
                peak_index(a, k.suc) < peak_after(a, peak_index(a, k.suc)) and
                    is_peak(a, peak_after(a, peak_index(a, k.suc)))
                peak_index(a, k.suc.suc) = peak_after(a, peak_index(a, k.suc))
                peak_index(a, k.suc) < peak_index(a, k.suc.suc)
                p(k.suc)
            }
        }
        p(Nat.0) and forall(k: Nat) { p(k) implies p(k.suc) }
        Nat.induction(p)
        forall(k: Nat) { p(k) }
        p(n)
        peak_index(a, n) < peak_index(a, n.suc)
    }
}

/// Under unbounded peaks, each selected value is at least the next selected
/// value.
theorem peak_index_later_le_at(a: Nat -> Real, n: Nat) {
    (forall(m: Nat) { exists(p: Nat) { m < p and is_peak(a, p) } })
    implies a(peak_index(a, n.suc)) <= a(peak_index(a, n))
} by {
    if forall(m: Nat) { exists(p: Nat) { m < p and is_peak(a, p) } } {
        define p(k: Nat) -> Bool {
            is_peak(a, peak_index(a, k)) and peak_index(a, k) < peak_index(a, k.suc)
        }
        exists(pk: Nat) { Nat.0 < pk and is_peak(a, pk) }
        peak_after_spec(a, Nat.0)
        Nat.0 < peak_after(a, Nat.0) and is_peak(a, peak_after(a, Nat.0))
        is_peak(a, peak_after(a, Nat.0))
        exists(pk: Nat) { peak_after(a, Nat.0) < pk and is_peak(a, pk) }
        peak_after_spec(a, peak_after(a, Nat.0))
        peak_after(a, Nat.0) < peak_after(a, peak_after(a, Nat.0)) and
            is_peak(a, peak_after(a, peak_after(a, Nat.0)))
        peak_index(a, Nat.0) = peak_after(a, Nat.0)
        is_peak(a, peak_index(a, Nat.0))
        peak_index(a, Nat.0.suc) = peak_after(a, peak_index(a, Nat.0))
        peak_index(a, Nat.0) < peak_index(a, Nat.0.suc)
        is_peak(a, peak_index(a, Nat.0)) and peak_index(a, Nat.0) < peak_index(a, Nat.0.suc)
        p(Nat.0)
        forall(k: Nat) {
            if p(k) {
                p(k) = (is_peak(a, peak_index(a, k)) and peak_index(a, k) < peak_index(a, k.suc))
                is_peak(a, peak_index(a, k))
                peak_index(a, k) < peak_index(a, k.suc)
                exists(pk: Nat) { peak_index(a, k) < pk and is_peak(a, pk) }
                peak_after_spec(a, peak_index(a, k))
                peak_index(a, k) < peak_after(a, peak_index(a, k)) and
                    is_peak(a, peak_after(a, peak_index(a, k)))
                peak_index(a, k.suc) = peak_after(a, peak_index(a, k))
                is_peak(a, peak_index(a, k.suc))
                exists(pk: Nat) { peak_index(a, k.suc) < pk and is_peak(a, pk) }
                peak_after_spec(a, peak_index(a, k.suc))
                peak_index(a, k.suc) < peak_after(a, peak_index(a, k.suc)) and
                    is_peak(a, peak_after(a, peak_index(a, k.suc)))
                peak_index(a, k.suc.suc) = peak_after(a, peak_index(a, k.suc))
                peak_index(a, k.suc) < peak_index(a, k.suc.suc)
                is_peak(a, peak_index(a, k.suc)) and peak_index(a, k.suc) < peak_index(a, k.suc.suc)
                p(k.suc)
            }
        }
        p(Nat.0) and forall(k: Nat) { p(k) implies p(k.suc) }
        Nat.induction(p)
        forall(k: Nat) { p(k) }
        p(n)
        is_peak(a, peak_index(a, n)) and peak_index(a, n) < peak_index(a, n.suc)
        is_peak(a, peak_index(a, n))
        peak_index(a, n) < peak_index(a, n.suc)
        is_peak(a, peak_index(a, n)) = forall(k: Nat) {
            peak_index(a, n) < k implies a(k) <= a(peak_index(a, n))
        }
        forall(k: Nat) { peak_index(a, n) < k implies a(k) <= a(peak_index(a, n)) }
        (peak_index(a, n) < peak_index(a, n.suc) implies
            a(peak_index(a, n.suc)) <= a(peak_index(a, n)))
        a(peak_index(a, n.suc)) <= a(peak_index(a, n))
    }
}

/// Under eventually-no-peaks, the strict index map strictly increases at
/// each step.
theorem strict_index_strict_at(a: Nat -> Real, m0: Nat, n: Nat) {
    (forall(p: Nat) { m0 < p implies not is_peak(a, p) })
    implies strict_index(a, m0.suc, n) < strict_index(a, m0.suc, n.suc)
} by {
    if forall(p: Nat) { m0 < p implies not is_peak(a, p) } {
        define p(k: Nat) -> Bool {
            m0.suc <= strict_index(a, m0.suc, k) and
            strict_index(a, m0.suc, k) < strict_index(a, m0.suc, k.suc)
        }
        strict_index(a, m0.suc, Nat.0) = m0.suc
        lte_ref(m0.suc)
        m0.suc <= strict_index(a, m0.suc, Nat.0)
        lt_suc(m0)
        m0 < m0.suc
        m0 < m0.suc implies not is_peak(a, m0.suc)
        not is_peak(a, m0.suc)
        not_peak_imp_later_strict(a, m0.suc)
        exists(k: Nat) { m0.suc < k and a(m0.suc) < a(k) }
        strict_after_spec(a, m0.suc)
        m0.suc < strict_after(a, m0.suc) and a(m0.suc) < a(strict_after(a, m0.suc))
        strict_index(a, m0.suc, Nat.0.suc) = strict_after(a, strict_index(a, m0.suc, Nat.0))
        strict_index(a, m0.suc, Nat.0) < strict_index(a, m0.suc, Nat.0.suc)
        m0.suc <= strict_index(a, m0.suc, Nat.0) and
            strict_index(a, m0.suc, Nat.0) < strict_index(a, m0.suc, Nat.0.suc)
        p(Nat.0)
        forall(k: Nat) {
            if p(k) {
                p(k) = (m0.suc <= strict_index(a, m0.suc, k) and
                    strict_index(a, m0.suc, k) < strict_index(a, m0.suc, k.suc))
                m0.suc <= strict_index(a, m0.suc, k)
                strict_index(a, m0.suc, k) < strict_index(a, m0.suc, k.suc)
                lte_and_lt(m0.suc, strict_index(a, m0.suc, k), strict_index(a, m0.suc, k.suc))
                m0.suc < strict_index(a, m0.suc, k.suc)
                lt_imp_lte(m0.suc, strict_index(a, m0.suc, k.suc))
                m0.suc <= strict_index(a, m0.suc, k.suc)
                lt_suc(m0)
                m0 < m0.suc
                lt_and_lte(m0, m0.suc, strict_index(a, m0.suc, k.suc))
                m0 < strict_index(a, m0.suc, k.suc)
                m0 < strict_index(a, m0.suc, k.suc) implies not is_peak(a, strict_index(a, m0.suc, k.suc))
                not is_peak(a, strict_index(a, m0.suc, k.suc))
                not_peak_imp_later_strict(a, strict_index(a, m0.suc, k.suc))
                exists(z: Nat) {
                    strict_index(a, m0.suc, k.suc) < z and
                    a(strict_index(a, m0.suc, k.suc)) < a(z)
                }
                strict_after_spec(a, strict_index(a, m0.suc, k.suc))
                strict_index(a, m0.suc, k.suc) < strict_after(a, strict_index(a, m0.suc, k.suc)) and
                    a(strict_index(a, m0.suc, k.suc)) < a(strict_after(a, strict_index(a, m0.suc, k.suc)))
                strict_index(a, m0.suc, k.suc.suc) =
                    strict_after(a, strict_index(a, m0.suc, k.suc))
                strict_index(a, m0.suc, k.suc) < strict_index(a, m0.suc, k.suc.suc)
                m0.suc <= strict_index(a, m0.suc, k.suc) and
                    strict_index(a, m0.suc, k.suc) < strict_index(a, m0.suc, k.suc.suc)
                p(k.suc)
            }
        }
        p(Nat.0) and forall(k: Nat) { p(k) implies p(k.suc) }
        Nat.induction(p)
        forall(k: Nat) { p(k) }
        p(n)
        m0.suc <= strict_index(a, m0.suc, n) and
            strict_index(a, m0.suc, n) < strict_index(a, m0.suc, n.suc)
        strict_index(a, m0.suc, n) < strict_index(a, m0.suc, n.suc)
    }
}

/// Under eventually-no-peaks, the selected values strictly increase at each
/// step.
theorem strict_index_incr_at(a: Nat -> Real, m0: Nat, n: Nat) {
    (forall(p: Nat) { m0 < p implies not is_peak(a, p) })
    implies a(strict_index(a, m0.suc, n)) < a(strict_index(a, m0.suc, n.suc))
} by {
    if forall(p: Nat) { m0 < p implies not is_peak(a, p) } {
        define p(k: Nat) -> Bool {
            m0.suc <= strict_index(a, m0.suc, k) and
            strict_index(a, m0.suc, k) < strict_index(a, m0.suc, k.suc) and
            a(strict_index(a, m0.suc, k)) < a(strict_index(a, m0.suc, k.suc))
        }
        strict_index(a, m0.suc, Nat.0) = m0.suc
        lte_ref(m0.suc)
        m0.suc <= strict_index(a, m0.suc, Nat.0)
        lt_suc(m0)
        m0 < m0.suc
        m0 < m0.suc implies not is_peak(a, m0.suc)
        not is_peak(a, m0.suc)
        not_peak_imp_later_strict(a, m0.suc)
        exists(k: Nat) { m0.suc < k and a(m0.suc) < a(k) }
        strict_after_spec(a, m0.suc)
        m0.suc < strict_after(a, m0.suc) and a(m0.suc) < a(strict_after(a, m0.suc))
        strict_index(a, m0.suc, Nat.0.suc) = strict_after(a, strict_index(a, m0.suc, Nat.0))
        strict_index(a, m0.suc, Nat.0) < strict_index(a, m0.suc, Nat.0.suc)
        a(strict_index(a, m0.suc, Nat.0)) < a(strict_index(a, m0.suc, Nat.0.suc))
        m0.suc <= strict_index(a, m0.suc, Nat.0) and
            strict_index(a, m0.suc, Nat.0) < strict_index(a, m0.suc, Nat.0.suc) and
            a(strict_index(a, m0.suc, Nat.0)) < a(strict_index(a, m0.suc, Nat.0.suc))
        p(Nat.0)
        forall(k: Nat) {
            if p(k) {
                p(k) = (m0.suc <= strict_index(a, m0.suc, k) and
                    strict_index(a, m0.suc, k) < strict_index(a, m0.suc, k.suc) and
                    a(strict_index(a, m0.suc, k)) < a(strict_index(a, m0.suc, k.suc)))
                m0.suc <= strict_index(a, m0.suc, k)
                strict_index(a, m0.suc, k) < strict_index(a, m0.suc, k.suc)
                a(strict_index(a, m0.suc, k)) < a(strict_index(a, m0.suc, k.suc))
                lte_and_lt(m0.suc, strict_index(a, m0.suc, k), strict_index(a, m0.suc, k.suc))
                m0.suc < strict_index(a, m0.suc, k.suc)
                lt_imp_lte(m0.suc, strict_index(a, m0.suc, k.suc))
                m0.suc <= strict_index(a, m0.suc, k.suc)
                lt_suc(m0)
                m0 < m0.suc
                lt_and_lte(m0, m0.suc, strict_index(a, m0.suc, k.suc))
                m0 < strict_index(a, m0.suc, k.suc)
                m0 < strict_index(a, m0.suc, k.suc) implies not is_peak(a, strict_index(a, m0.suc, k.suc))
                not is_peak(a, strict_index(a, m0.suc, k.suc))
                not_peak_imp_later_strict(a, strict_index(a, m0.suc, k.suc))
                exists(z: Nat) {
                    strict_index(a, m0.suc, k.suc) < z and
                    a(strict_index(a, m0.suc, k.suc)) < a(z)
                }
                strict_after_spec(a, strict_index(a, m0.suc, k.suc))
                strict_index(a, m0.suc, k.suc) < strict_after(a, strict_index(a, m0.suc, k.suc)) and
                    a(strict_index(a, m0.suc, k.suc)) < a(strict_after(a, strict_index(a, m0.suc, k.suc)))
                strict_index(a, m0.suc, k.suc.suc) =
                    strict_after(a, strict_index(a, m0.suc, k.suc))
                strict_index(a, m0.suc, k.suc) < strict_index(a, m0.suc, k.suc.suc)
                a(strict_index(a, m0.suc, k.suc)) < a(strict_index(a, m0.suc, k.suc.suc))
                m0.suc <= strict_index(a, m0.suc, k.suc) and
                    strict_index(a, m0.suc, k.suc) < strict_index(a, m0.suc, k.suc.suc) and
                    a(strict_index(a, m0.suc, k.suc)) < a(strict_index(a, m0.suc, k.suc.suc))
                p(k.suc)
            }
        }
        p(Nat.0) and forall(k: Nat) { p(k) implies p(k.suc) }
        Nat.induction(p)
        forall(k: Nat) { p(k) }
        p(n)
        m0.suc <= strict_index(a, m0.suc, n) and
            strict_index(a, m0.suc, n) < strict_index(a, m0.suc, n.suc) and
            a(strict_index(a, m0.suc, n)) < a(strict_index(a, m0.suc, n.suc))
        a(strict_index(a, m0.suc, n)) < a(strict_index(a, m0.suc, n.suc))
    }
}

/// A strictly increasing natural sequence is nondecreasing between two
/// indices.
theorem strict_increasing_nat_distant(f: Nat -> Nat, m: Nat, n: Nat) {
    (forall(n0: Nat) { f(n0) < f(n0.suc) }) and m <= n implies f(m) <= f(n)
} by {
    if forall(n0: Nat) { f(n0) < f(n0.suc) } and m <= n {
        define p(k: Nat) -> Bool {
            forall(j: Nat) {
                j + k <= n implies f(j) <= f(j + k)
            }
        }
        forall(j: Nat) {
            if j + Nat.0 <= n {
                j + Nat.0 = j
                lte_ref(f(j))
                f(j) <= f(j)
            }
        }
        p(Nat.0)
        forall(k: Nat) {
            if p(k) {
                p(k) = forall(j: Nat) {
                    j + k <= n implies f(j) <= f(j + k)
                }
                forall(j: Nat) {
                    if j + k.suc <= n {
                        j + k.suc = (j + k).suc
                        (j + k).suc <= n
                        lt_suc(j + k)
                        j + k < (j + k).suc
                        lt_imp_lte(j + k, (j + k).suc)
                        j + k <= (j + k).suc
                        lte_trans(j + k, (j + k).suc, n)
                        j + k <= n
                        f(j) <= f(j + k)
                        f(j + k) < f((j + k).suc)
                        lt_imp_lte(f(j + k), f((j + k).suc))
                        f(j + k) <= f((j + k).suc)
                        lte_trans(f(j), f(j + k), f((j + k).suc))
                        f(j) <= f((j + k).suc)
                        (j + k).suc = j + k.suc
                        f(j) <= f(j + k.suc)
                    }
                }
                p(k.suc)
            }
        }
        p(Nat.0) and forall(k: Nat) { p(k) implies p(k.suc) }
        Nat.induction(p)
        let k: Nat satisfy {
            m + k = n
        }
        p(k)
        p(k) = forall(j: Nat) {
            j + k <= n implies f(j) <= f(j + k)
        }
        m + k = n
        lte_ref(n)
        n <= n
        m + k <= n
        f(m) <= f(m + k)
        m + k = n
        f(m) <= f(n)
    }
}

/// A strictly increasing natural sequence is nondecreasing.
theorem strict_increasing_nat_is_nondecreasing(f: Nat -> Nat) {
    (forall(n: Nat) { f(n) < f(n.suc) })
    implies is_nondecreasing_nat(f)
} by {
    if forall(n: Nat) { f(n) < f(n.suc) } {
        forall(i: Nat, j: Nat) {
            if i <= j {
                strict_increasing_nat_distant(f, i, j)
                f(i) <= f(j)
            }
        }
        is_nondecreasing_nat(f) = forall(i: Nat, j: Nat) {
            i <= j implies f(i) <= f(j)
        }
        is_nondecreasing_nat(f)
    }
}

/// A strictly increasing natural sequence is monotone.
theorem strict_increasing_nat_is_monotone(f: Nat -> Nat) {
    (forall(n: Nat) { f(n) < f(n.suc) }) implies is_monotone(f)
} by {
    if forall(n: Nat) { f(n) < f(n.suc) } {
        strict_increasing_nat_is_nondecreasing(f)
        is_nondecreasing_nat(f) = forall(i: Nat, j: Nat) {
            i <= j implies f(i) <= f(j)
        }
        monotone_from_forall(f)
        is_monotone(f)
    }
}

/// A strictly increasing natural sequence is at least `f(0) + n` at `n`.
theorem strict_increasing_nat_gte_base_plus(f: Nat -> Nat, n: Nat) {
    (forall(n0: Nat) { f(n0) < f(n0.suc) })
    implies f(Nat.0) + n <= f(n)
} by {
    if forall(n0: Nat) { f(n0) < f(n0.suc) } {
        define p(k: Nat) -> Bool {
            f(Nat.0) + k <= f(k)
        }
        f(Nat.0) + Nat.0 = f(Nat.0)
        lte_ref(f(Nat.0))
        p(Nat.0)
        forall(k: Nat) {
            if p(k) {
                f(Nat.0) + k <= f(k)
                f(k) < f(k.suc)
                lte_suc_suc(f(Nat.0) + k, f(k))
                (f(Nat.0) + k).suc <= f(k).suc
                lt_imp_lte_suc(f(k), f(k.suc))
                f(k).suc <= f(k.suc)
                add_suc_right(f(Nat.0), k)
                f(Nat.0) + k.suc = (f(Nat.0) + k).suc
                lte_trans(f(Nat.0) + k.suc, f(k).suc, f(k.suc))
                f(Nat.0) + k.suc <= f(k.suc)
                p(k.suc)
            }
        }
        p(Nat.0) and forall(k: Nat) { p(k) implies p(k.suc) }
        Nat.induction(p)
        p(n)
        f(Nat.0) + n <= f(n)
    }
}

/// A strictly increasing natural sequence is unbounded.
theorem strict_increasing_nat_is_unbounded(f: Nat -> Nat) {
    (forall(n: Nat) { f(n) < f(n.suc) }) implies is_unbounded(f)
} by {
    if forall(n: Nat) { f(n) < f(n.suc) } {
        forall(bound: Nat) {
            strict_increasing_nat_gte_base_plus(f, bound.suc)
            f(Nat.0) + bound.suc <= f(bound.suc)
            lte_ref(bound.suc)
            bound.suc <= bound.suc
            lte_add_left(bound.suc, Nat.0, f(Nat.0))
            Nat.0 <= f(Nat.0) implies bound.suc + Nat.0 <= bound.suc + f(Nat.0)
            nat_zero_lte(f(Nat.0))
            Nat.0 <= f(Nat.0)
            bound.suc + Nat.0 <= bound.suc + f(Nat.0)
            bound.suc + Nat.0 = bound.suc
            bound.suc <= f(Nat.0) + bound.suc
            lte_trans(bound.suc, f(Nat.0) + bound.suc, f(bound.suc))
            bound.suc <= f(bound.suc)
            lt_suc(bound)
            bound < bound.suc
            lt_and_lte(bound, bound.suc, f(bound.suc))
            bound < f(bound.suc)
            exists(n: Nat) { bound < f(n) }
        }
        is_unbounded(f)
    }
}

/// Under unbounded peaks, the peak-index map is a subsequence index and
/// selects a nonincreasing subsequence.
theorem unbounded_peaks_subsequence(a: Nat -> Real) {
    (forall(m: Nat) { exists(p: Nat) { m < p and is_peak(a, p) } })
    implies exists(f: Nat -> Nat) {
        is_subsequence_index(f) and is_decreasing_seq(subsequence(a, f))
    }
} by {
    if forall(m: Nat) { exists(p: Nat) { m < p and is_peak(a, p) } } {
        // the peak-index map is a subsequence index
        forall(n: Nat) {
            peak_index_strict_at(a, n)
            (forall(m: Nat) { exists(p: Nat) { m < p and is_peak(a, p) } } implies
                peak_index(a, n) < peak_index(a, n.suc))
            peak_index(a, n) < peak_index(a, n.suc)
        }
        strict_increasing_nat_is_monotone(peak_index(a))
        is_monotone(peak_index(a))
        forall(n: Nat) {
            peak_index_strict_at(a, n)
            (forall(m: Nat) { exists(p: Nat) { m < p and is_peak(a, p) } } implies
                peak_index(a, n) < peak_index(a, n.suc))
            peak_index(a, n) < peak_index(a, n.suc)
        }
        strict_increasing_nat_is_unbounded(peak_index(a))
        is_unbounded(peak_index(a))
        subsequence_index_of_monotone_unbounded(peak_index(a))
        is_subsequence_index(peak_index(a))
        // the selected subsequence is nonincreasing
        forall(n: Nat) {
            peak_index_later_le_at(a, n)
            (forall(m: Nat) { exists(p: Nat) { m < p and is_peak(a, p) } } implies
                a(peak_index(a, n.suc)) <= a(peak_index(a, n)))
            a(peak_index(a, n.suc)) <= a(peak_index(a, n))
            subsequence(a, peak_index(a), n.suc) = a(peak_index(a, n.suc))
            subsequence(a, peak_index(a), n) = a(peak_index(a, n))
            subsequence(a, peak_index(a), n.suc) <= subsequence(a, peak_index(a), n)
        }
        is_decreasing_seq(subsequence(a, peak_index(a)))
        is_subsequence_index(peak_index(a)) and is_decreasing_seq(subsequence(a, peak_index(a)))
        exists(f: Nat -> Nat) {
            is_subsequence_index(f) and is_decreasing_seq(subsequence(a, f))
        }
    }
}

/// Under eventually-no-peaks, the strict index map is a subsequence index
/// and selects an increasing subsequence.
theorem no_peaks_subsequence(a: Nat -> Real) {
    (exists(m0: Nat) { forall(p: Nat) { m0 < p implies not is_peak(a, p) } })
    implies exists(f: Nat -> Nat) {
        is_subsequence_index(f) and is_increasing(subsequence(a, f))
    }
} by {
    if exists(m0: Nat) { forall(p: Nat) { m0 < p implies not is_peak(a, p) } } {
        let m0: Nat satisfy {
            forall(p: Nat) { m0 < p implies not is_peak(a, p) }
        }
        // the strict index map is a subsequence index
        forall(n: Nat) {
            strict_index_strict_at(a, m0, n)
            (forall(p: Nat) { m0 < p implies not is_peak(a, p) } implies
                strict_index(a, m0.suc, n) < strict_index(a, m0.suc, n.suc))
            strict_index(a, m0.suc, n) < strict_index(a, m0.suc, n.suc)
        }
        strict_increasing_nat_is_monotone(strict_index(a, m0.suc))
        is_monotone(strict_index(a, m0.suc))
        forall(n: Nat) {
            strict_index_strict_at(a, m0, n)
            (forall(p: Nat) { m0 < p implies not is_peak(a, p) } implies
                strict_index(a, m0.suc, n) < strict_index(a, m0.suc, n.suc))
            strict_index(a, m0.suc, n) < strict_index(a, m0.suc, n.suc)
        }
        strict_increasing_nat_is_unbounded(strict_index(a, m0.suc))
        is_unbounded(strict_index(a, m0.suc))
        subsequence_index_of_monotone_unbounded(strict_index(a, m0.suc))
        is_subsequence_index(strict_index(a, m0.suc))
        // the selected subsequence is increasing
        forall(n: Nat) {
            strict_index_incr_at(a, m0, n)
            (forall(p: Nat) { m0 < p implies not is_peak(a, p) } implies
                a(strict_index(a, m0.suc, n)) < a(strict_index(a, m0.suc, n.suc)))
            a(strict_index(a, m0.suc, n)) < a(strict_index(a, m0.suc, n.suc))
            lt_imp_lte(a(strict_index(a, m0.suc, n)), a(strict_index(a, m0.suc, n.suc)))
            a(strict_index(a, m0.suc, n)) <= a(strict_index(a, m0.suc, n.suc))
            subsequence(a, strict_index(a, m0.suc), n) = a(strict_index(a, m0.suc, n))
            subsequence(a, strict_index(a, m0.suc), n.suc) = a(strict_index(a, m0.suc, n.suc))
            subsequence(a, strict_index(a, m0.suc), n) <= subsequence(a, strict_index(a, m0.suc), n.suc)
        }
        is_increasing(subsequence(a, strict_index(a, m0.suc)))
        is_subsequence_index(strict_index(a, m0.suc)) and
            is_increasing(subsequence(a, strict_index(a, m0.suc)))
        exists(f: Nat -> Nat) {
            is_subsequence_index(f) and is_increasing(subsequence(a, f))
        }
    }
}

/// The Bolzano–Weierstrass theorem: every bounded sequence of real numbers
/// has a convergent subsequence.
theorem bolzano_weierstrass(a: Nat -> Real) {
    exists(lo: Real, hi: Real) {
        forall(n: Nat) { lo <= a(n) and a(n) <= hi }
    } implies
        exists(f: Nat -> Nat, l: Real) {
            is_monotone(f) and is_unbounded(f) and converges_to(subsequence(a, f), l)
        }
} by {
    if exists(lo: Real, hi: Real) {
        forall(n: Nat) { lo <= a(n) and a(n) <= hi }
    } {
        let lo: Real satisfy {
            exists(hi: Real) {
                forall(n: Nat) { lo <= a(n) and a(n) <= hi }
            }
        }
        let hi: Real satisfy {
            forall(n: Nat) { lo <= a(n) and a(n) <= hi }
        }
        if forall(m: Nat) { exists(p: Nat) { m < p and is_peak(a, p) } } {
            // Case 1: unbounded peaks give a nonincreasing subsequence,
            // bounded below by lo, hence convergent.
            unbounded_peaks_subsequence(a)
            exists(f: Nat -> Nat) {
                is_subsequence_index(f) and is_decreasing_seq(subsequence(a, f))
            }
            let f: Nat -> Nat satisfy {
                is_subsequence_index(f) and is_decreasing_seq(subsequence(a, f))
            }
            subsequence_index_is_monotone(f)
            is_monotone(f)
            subsequence_index_is_unbounded(f)
            is_unbounded(f)
            is_decreasing_seq(subsequence(a, f))
            // bounded below by lo
            forall(n: Nat) {
                forall(n0: Nat) { lo <= a(n0) and a(n0) <= hi }
                lo <= a(f(n))
                subsequence(a, f, n) = a(f(n))
                lo <= subsequence(a, f, n)
            }
            is_lower_bound(subsequence(a, f), lo)
            // negate: increasing and bounded above by -lo
            decreasing_neg_increasing(subsequence(a, f))
            is_increasing(neg_seq(subsequence(a, f)))
            forall(n: Nat) {
                forall(n0: Nat) { lo <= a(n0) and a(n0) <= hi }
                lo <= a(f(n))
                subsequence(a, f, n) = a(f(n))
                lo <= subsequence(a, f, n)
                lte_imp_neg_lte_neg(lo, subsequence(a, f, n))
                -subsequence(a, f, n) <= -lo
                neg_seq(subsequence(a, f), n) = -subsequence(a, f, n)
                neg_seq(subsequence(a, f), n) <= -lo
            }
            is_upper_bound(neg_seq(subsequence(a, f)), -lo)
            monotone_convergence_principle(neg_seq(subsequence(a, f)), -lo)
            converges(neg_seq(subsequence(a, f)))
            neg_seq_converges_converse(subsequence(a, f))
            converges(subsequence(a, f))
            convergent_converges_to_limit(subsequence(a, f))
            converges_to(subsequence(a, f), limit(subsequence(a, f)))
            is_monotone(f) and is_unbounded(f) and converges_to(subsequence(a, f), limit(subsequence(a, f)))
            exists(g: Nat -> Nat, l: Real) {
                is_monotone(g) and is_unbounded(g) and converges_to(subsequence(a, g), l)
            }
        } else {
            // Case 2: finitely many peaks give an increasing subsequence,
            // bounded above by hi, hence convergent.
            not forall(m: Nat) { exists(p: Nat) { m < p and is_peak(a, p) } }
            exists(m0: Nat) {
                not exists(p: Nat) { m0 < p and is_peak(a, p) }
            }
            let m0: Nat satisfy {
                not exists(p: Nat) { m0 < p and is_peak(a, p) }
            }
            forall(p: Nat) {
                if m0 < p {
                    not exists(z: Nat) { m0 < z and is_peak(a, z) }
                    not (m0 < p and is_peak(a, p))
                    m0 < p
                    not is_peak(a, p)
                }
            }
            exists(m1: Nat) { forall(p: Nat) { m1 < p implies not is_peak(a, p) } }
            no_peaks_subsequence(a)
            exists(f: Nat -> Nat) {
                is_subsequence_index(f) and is_increasing(subsequence(a, f))
            }
            let f: Nat -> Nat satisfy {
                is_subsequence_index(f) and is_increasing(subsequence(a, f))
            }
            subsequence_index_is_monotone(f)
            is_monotone(f)
            subsequence_index_is_unbounded(f)
            is_unbounded(f)
            is_increasing(subsequence(a, f))
            forall(n: Nat) {
                forall(n0: Nat) { lo <= a(n0) and a(n0) <= hi }
                a(f(n)) <= hi
                subsequence(a, f, n) = a(f(n))
                subsequence(a, f, n) <= hi
            }
            is_upper_bound(subsequence(a, f), hi)
            monotone_convergence_principle(subsequence(a, f), hi)
            converges(subsequence(a, f))
            convergent_converges_to_limit(subsequence(a, f))
            converges_to(subsequence(a, f), limit(subsequence(a, f)))
            is_monotone(f) and is_unbounded(f) and converges_to(subsequence(a, f), limit(subsequence(a, f)))
            exists(g: Nat -> Nat, l: Real) {
                is_monotone(g) and is_unbounded(g) and converges_to(subsequence(a, g), l)
            }
        }
        exists(f: Nat -> Nat, l: Real) {
            is_monotone(f) and is_unbounded(f) and converges_to(subsequence(a, f), l)
        }
    }
}

