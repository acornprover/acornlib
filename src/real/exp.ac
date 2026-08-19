from nat import Nat, pow_zero, semiring_zero_pow, from_nat, from_nat_add, from_nat_mul, lt_and_lte, lte_and_lt, lt_imp_lte_suc, lt_or_lte, lt_add_suc, nat_mul_3_2, nat_mul_4_6, pow_one, one_pow, factorial_zero, factorial_one, factorial_step, add_sub, alt_induction
from int import Int, lt_from_nat
from order import gt_of_gte_of_gt
from rat import Rat, from_nat_nonneg
from real.real_field import Real, mul_div, div_le_of_mul_le, mul_le_mul_pos_right, mul_le_mul_pos_left, inverse_div, div_mul_cancel_right, mul_left_cancel, mul_div_cancel
from real.real_ring import converges, limit, mul_abs, from_nat_is_from_rat, lte_mul_nonneg_right, mul_from_rat, rat_from_nat_add, mul_nonneg, lt_mul_pos_left
from real.real_seq import converges_to, eventual_lb, lb_lte_limit, eventual_ub, ub_imp_limit_lte
from real.real_base import abs_from_rat, real_is_transitive, one_half_positive, one_half_plus_one_half, from_rat_maintains_lte, from_rat_maintains_lt, add_lt_lt, lt_trans, lt_add_right, add_lte_add, add_zero_right, lte_trans, lte_add_left, lte_add_right, lte_lt_trans, lt_add_pos, lt_lte_trans
from real.abs_conv import absolutely_converges, abs_fn, absolutely_converges_comparison
from real.real_series import seq_lte, abs_pow, tail, partial_tail_conv_imp_partial_conv, comparison_test, const_converges, mul_seq, is_lower_bound, geom_converges, partial_monotone, nonneg_partial_increasing, is_increasing, distant_increasing, increasing_convergent_bounded_by_limit, is_upper_bound, partial_tail, partial_sums_seq_lte, partial_mul_seq_comm, pos_geom_indirect_upper_bound, partial_suc, partial_zero, partial_one, pow_nonneg
from real.cauchy import cauchy_product, cauchy_seq, cauchy_product_converges, cauchy_coefficient
from list import partial, sum, map
from list import partial_scalar_mul
from algebra.semigroup import mul_fn
from algebra.add_semigroup import add_fn
from data.basic.functions import compose
from data.basic.relation_basic import is_transitive
from combinatorics import binom
from comm_ring import binomial_term, binomial
from ordered_field import inverse_on_positive_flips_inequality

numerals Real
numerals Nat

/// Division function: returns a function that divides each value by a constant.
/// Similar to mul_fn, but for division.
define div_fn(f: Nat -> Real, c: Real) -> (Nat -> Real) {
    function(k: Nat) { f(k) / c }
}

/// The nth term in the Taylor series expansion of e^x.
/// For a real number x and natural number n, this returns x^n / n!
define exp_term(x: Real, n: Nat) -> Real {
    x.pow(n) / Real.from_rat(Rat.from_nat(n.factorial))
}

// Helper lemmas for working with exp_term

/// Factorial is positive for all n.
theorem factorial_pos(n: Nat) {
    Real.from_rat(Rat.from_nat(n.factorial)) > Real.0
} by {
    // 1 <= n.factorial for all n
    // So Rat.1 <= Rat.from_nat(n.factorial)
    // And Real.1 <= Real.from_rat(Rat.from_nat(n.factorial))
    // Therefore Real.from_rat(Rat.from_nat(n.factorial)) > Real.0
    Nat.1 <= n.factorial
    Rat.1 <= Rat.from_nat(n.factorial)
    Real.from_rat(Rat.1) <= Real.from_rat(Rat.from_nat(n.factorial))
}

/// The absolute value of the inverse equals the inverse of the absolute value.
theorem abs_inverse(b: Real) {
    b != Real.0 implies b.inverse.abs = b.abs.inverse
} by {
    if b != Real.0 {
        b.abs != Real.0
        // |b| * |1/b| = |b * 1/b| = |1| = 1
        // So |1/b| = 1 / |b|
        b.abs * b.inverse.abs = (b * b.inverse).abs
        (b * b.inverse).abs = Real.1.abs
        Real.1.abs = Real.1
        b.abs * b.inverse.abs = Real.1
        b.inverse.abs = b.abs.inverse
    }
}

/// The absolute value of a division equals the division of absolute values.
theorem abs_div(a: Real, b: Real) {
    b != Real.0 implies (a / b).abs = a.abs / b.abs
} by {
    if b != Real.0 {
        (a / b).abs = (a * b.inverse).abs
        (a * b.inverse).abs = a.abs * b.inverse.abs
        b.inverse.abs = b.abs.inverse
        a.abs * b.inverse.abs = a.abs * b.abs.inverse
        a.abs * b.abs.inverse = a.abs / b.abs
    }
}

/// Converting factorial from Nat to Rat to Real for successor.
theorem factorial_suc_real(n: Nat) {
    Real.from_rat(Rat.from_nat(n.suc.factorial)) =
    Real.from_rat(Rat.from_nat(n.suc)) * Real.from_rat(Rat.from_nat(n.factorial))
} by {
    // n.suc.factorial = n.suc * n.factorial (by factorial_step)
    // Rat.from_nat is defined as Rat.from_int ∘ Int.from_nat
    // Int.from_nat(a * b) = Int.from_nat(a) * Int.from_nat(b)
    // Rat.from_int and Real.from_rat both preserve multiplication
    n.suc.factorial = n.suc * n.factorial
    Rat.from_nat(n.suc.factorial) = Rat.from_nat(n.suc * n.factorial)

    // Expand using definition: Rat.from_nat(x) = Rat.from_int(Int.from_nat(x))
    Rat.from_nat(n.suc * n.factorial) = Rat.from_int(Int.from_nat(n.suc * n.factorial))
    Int.from_nat(n.suc * n.factorial) = Int.from_nat(n.suc) * Int.from_nat(n.factorial)
    Rat.from_int(Int.from_nat(n.suc) * Int.from_nat(n.factorial)) =
        Rat.from_int(Int.from_nat(n.suc)) * Rat.from_int(Int.from_nat(n.factorial))

    Rat.from_nat(n.suc) = Rat.from_int(Int.from_nat(n.suc))
    Rat.from_nat(n.factorial) = Rat.from_int(Int.from_nat(n.factorial))

    Real.from_rat(Rat.from_nat(n.suc) * Rat.from_nat(n.factorial)) =
        Real.from_rat(Rat.from_nat(n.suc)) * Real.from_rat(Rat.from_nat(n.factorial))
}

/// Power of successor: x^(n+1) = x * x^n
theorem pow_suc(x: Real, n: Nat) {
    x.pow(n.suc) = x * x.pow(n)
}

/// n.suc converted to Real is positive.
theorem suc_pos(n: Nat) {
    Real.from_rat(Rat.from_nat(n.suc)) > Real.0
} by {
    // n.suc >= 1, so Rat.from_nat(n.suc) >= Rat.1 > Rat.0
    Nat.1 <= n.suc
    Rat.1 <= Rat.from_nat(n.suc)
    Real.from_rat(Rat.1) <= Real.from_rat(Rat.from_nat(n.suc))
}

/// The absolute value of exp_term(x, n).
/// |x^n / n!| = |x|^n / n! since n! is always positive.
theorem exp_term_abs(x: Real, n: Nat) {
    exp_term(x, n).abs = x.abs.pow(n) / Real.from_rat(Rat.from_nat(n.factorial))
} by {
    // exp_term(x, n) = x.pow(n) / Real.from_rat(Rat.from_nat(n.factorial))
    // |exp_term(x, n)| = |x.pow(n)| / |Real.from_rat(Rat.from_nat(n.factorial))| by abs_div
    // = |x|.pow(n) / |Real.from_rat(Rat.from_nat(n.factorial))| by abs_pow
    // = |x|.pow(n) / Real.from_rat(Rat.from_nat(n.factorial)) since factorial is positive

    let denom = Real.from_rat(Rat.from_nat(n.factorial))
    denom > Real.0
    denom != Real.0

    exp_term(x, n).abs = (x.pow(n) / denom).abs
    (x.pow(n) / denom).abs = x.pow(n).abs / denom.abs
    x.pow(n).abs = x.abs.pow(n)

    // Need to show denom.abs = denom (since denom > 0)
    not denom.is_negative
    denom.abs = denom
}

/// Multiplication form of the exp_term recurrence relation.
/// exp_term(x, n+1) * (n+1) = x * exp_term(x, n)
///
/// This is easier to prove than the division form, avoiding complex division algebra.
theorem exp_term_mul_recurrence(x: Real, n: Nat) {
    exp_term(x, n.suc) * Real.from_rat(Rat.from_nat(n.suc)) = x * exp_term(x, n)
} by {
    // Expand definitions
    exp_term(x, n.suc) = x.pow(n.suc) / Real.from_rat(Rat.from_nat(n.suc.factorial))
    exp_term(x, n) = x.pow(n) / Real.from_rat(Rat.from_nat(n.factorial))

    let f_n = Real.from_rat(Rat.from_nat(n.factorial))
    let f_n_suc = Real.from_rat(Rat.from_nat(n.suc.factorial))
    let n_plus_1 = Real.from_rat(Rat.from_nat(n.suc))

    // Use pow_suc and factorial_suc_real
    x.pow(n.suc) = x * x.pow(n)
    f_n_suc = n_plus_1 * f_n

    // These are nonzero
    f_n != Real.0
    f_n_suc != Real.0
    n_plus_1 != Real.0

    // LHS: exp_term(x, n.suc) * n_plus_1
    //    = (x^(n+1) / f_(n+1)) * (n+1)
    //    = (x * x^n / (n+1 * f_n)) * (n+1)
    //    = (x * x^n) / f_n

    exp_term(x, n.suc) * n_plus_1 = (x.pow(n.suc) / f_n_suc) * n_plus_1
    (x.pow(n.suc) / f_n_suc) * n_plus_1 = ((x * x.pow(n)) / (n_plus_1 * f_n)) * n_plus_1

    // Use mul_div_cancel: c * (b / c) = b when c != 0
    n_plus_1 * f_n = f_n * n_plus_1
    f_n * (((x * x.pow(n)) / (n_plus_1 * f_n)) * n_plus_1) = ((x * x.pow(n)) / (n_plus_1 * f_n)) * (f_n * n_plus_1)
    f_n_suc * ((x * x.pow(n)) / f_n_suc) = x * x.pow(n)
    ((x * x.pow(n)) / (n_plus_1 * f_n)) * n_plus_1 = (x * x.pow(n)) / f_n

    // RHS: x * exp_term(x, n)
    //    = x * (x^n / f_n)
    //    = (x * x^n) / f_n

    x * exp_term(x, n) = x * (x.pow(n) / f_n)
    x * (x.pow(n) / f_n) = (x * x.pow(n)) / f_n
}

theorem exp_term_mul_recurrence_abs(x: Real, n: Nat) {
    exp_term(x, n.suc).abs * Real.from_rat(Rat.from_nat(n.suc)) = x.abs * exp_term(x, n).abs
} by {
    // Use exp_term_mul_recurrence: exp_term(x, n.suc) * (n+1) = x * exp_term(x, n)
    let n_plus_1 = Real.from_rat(Rat.from_nat(n.suc))
    exp_term(x, n.suc) * n_plus_1 = x * exp_term(x, n)

    // Take absolute value of both sides
    (exp_term(x, n.suc) * n_plus_1).abs = (x * exp_term(x, n)).abs

    // Use mul_abs: |a * b| = |a| * |b|
    (exp_term(x, n.suc) * n_plus_1).abs = exp_term(x, n.suc).abs * n_plus_1.abs
    (x * exp_term(x, n)).abs = x.abs * exp_term(x, n).abs

    // n_plus_1 is positive, so |n_plus_1| = n_plus_1
    n_plus_1 > Real.0
    not n_plus_1.is_negative
    n_plus_1.abs = n_plus_1

    exp_term(x, n.suc).abs * n_plus_1 = x.abs * exp_term(x, n).abs
}

/// The ratio of absolute values of consecutive exp_term values.
/// |exp_term(x, n+1)| / |exp_term(x, n)| = |x| / (n+1)
///
/// This lemma is key for applying ratio test arguments to show convergence.
theorem exp_term_ratio(x: Real, n: Nat) {
    exp_term(x, n).abs != Real.0 implies
    exp_term(x, n.suc).abs / exp_term(x, n).abs = x.abs / Real.from_rat(Rat.from_nat(n.suc))
} by {
    if exp_term(x, n).abs != Real.0 {
        let n_plus_1 = Real.from_rat(Rat.from_nat(n.suc))

        // From exp_term_mul_recurrence_abs:
        // |exp_term(x, n+1)| * (n+1) = |x| * |exp_term(x, n)|
        exp_term(x, n.suc).abs * n_plus_1 = x.abs * exp_term(x, n).abs

        // n_plus_1 is positive and nonzero
        n_plus_1 > Real.0
        n_plus_1.is_positive
        n_plus_1 != Real.0

        // exp_term(x, n).abs is nonzero by assumption
        exp_term(x, n).abs != Real.0

        // Use prod_eq_to_div_eq
        exp_term(x, n.suc).abs / exp_term(x, n).abs = x.abs / n_plus_1

        // Divide both sides by (n_plus_1 * |exp_term(x, n)|)
        // LHS: |exp_term(x, n+1)| * (n+1) / ((n+1) * |exp_term(x, n)|)
        //    = |exp_term(x, n+1)| / |exp_term(x, n)|
        // RHS: |x| * |exp_term(x, n)| / ((n+1) * |exp_term(x, n)|)
        //    = |x| / (n+1)

        exp_term(x, n.suc).abs * n_plus_1 / (n_plus_1 * exp_term(x, n).abs) =
            x.abs * exp_term(x, n).abs / (n_plus_1 * exp_term(x, n).abs)

        // Simplify LHS using commutativity and cancellation
        exp_term(x, n.suc).abs * n_plus_1 / (n_plus_1 * exp_term(x, n).abs) =
            exp_term(x, n.suc).abs / exp_term(x, n).abs

        // Simplify RHS using commutativity and cancellation
        x.abs * exp_term(x, n).abs / (n_plus_1 * exp_term(x, n).abs) =
            x.abs / n_plus_1

        exp_term(x, n.suc).abs / exp_term(x, n).abs = x.abs / n_plus_1
        exp_term(x, n.suc).abs / exp_term(x, n).abs = x.abs / Real.from_rat(Rat.from_nat(n.suc))
    }
}

/// For n >= 1, 0^n = 0.
theorem zero_pow_pos(n: Nat) {
    n >= Nat.1 implies Real.0.pow(n) = Real.0
} by {
    define p(k: Nat) -> Bool {
        k >= Nat.1 implies Real.0.pow(k) = Real.0
    }

    // Base case: k = 1
    Real.0.pow(Nat.1) = Real.0

    // Inductive step
    forall(k: Nat) {
        if p(k) and k >= Nat.1 {
            Real.0.pow(k.suc) = Real.0 * Real.0.pow(k)
            Real.0.pow(k.suc) = Real.0
        }
    }

    Real.0.pow(n) = Real.0 or n = Nat.0
    p(n)
}

/// For n >= 1, exp_term(0, n) = 0.
theorem exp_term_zero_vanishes(n: Nat) {
    n >= Nat.1 implies exp_term(Real.0, n) = Real.0
} by {
    if n >= Nat.1 {
        exp_term(Real.0, n) = Real.0.pow(n) / Real.from_rat(Rat.from_nat(n.factorial))
        Real.0.pow(n) = Real.0
        Real.0 / Real.from_rat(Rat.from_nat(n.factorial)) = Real.0
    }
}

/// For any positive real, there exists a natural number greater than it.
/// This is a form of the Archimedean property.
theorem exists_nat_gt(x: Real) {
    x.is_positive implies exists(n: Nat) {
        Real.from_rat(Rat.from_nat(n)) > x
    }
} by {
    if x.is_positive {
        // Find a rational r > x
        let r: Rat satisfy {
            Real.from_rat(r) > x
        }

        // Since x > 0 and Real.from_rat(r) > x, we have Real.from_rat(r) > 0
        Real.from_rat(r) > Real.0
        r > Rat.0
        r.is_positive

        // Find a natural number n such that Rat.from_nat(n) > r
        // We can use the floor function: let n = floor(r) + 2
        let n: Nat satisfy {
            Rat.from_nat(n) > r
        }

        Rat.from_nat(n) > r
        Real.from_rat(Rat.from_nat(n)) > Real.from_rat(r)
        Real.from_rat(Rat.from_nat(n)) > x
    }
}

/// Two as a real number.
let two = Real.1 + Real.1

/// Two is positive.
theorem two_positive {
    two.is_positive
} by {
    Real.1.is_positive
}

/// Two is not zero.
theorem two_nonzero {
    two != Real.0
} by {
    two.is_positive
}

/// The inverse of a positive real is positive.
theorem inverse_pos(c: Real) {
    c.is_positive implies c.inverse.is_positive
} by {
    if c.is_positive {
        c != Real.0
        c * c.inverse = Real.1
        Real.1.is_positive

        // If c > 0 and c * c.inverse = 1 > 0
        // then c.inverse > 0
        // (because if c.inverse <= 0, then c * c.inverse <= 0, contradicting = 1)
        if not c.inverse.is_positive {
            // Then c.inverse <= 0
            c.inverse <= Real.0
            c * c.inverse <= c * Real.0
            c * Real.0 = Real.0
            c * c.inverse <= Real.0
            Real.1 <= Real.0
            false
        }
    }
}

/// If a < b and c > 0, then a / c < b / c.
theorem div_lt_div_pos(a: Real, b: Real, c: Real) {
    a < b and c > Real.0
    implies
    a / c < b / c
} by {
    if a < b and c > Real.0 {
        c != Real.0
        c.inverse.is_positive

        // a < b, so a * c.inverse < b * c.inverse (multiply by positive)
        a * c.inverse < b * c.inverse
        a / c = a * c.inverse
        b / c = b * c.inverse
        a / c < b / c
    }
}

/// For any positive real x, there exists N such that x < N/2.
/// This follows from the Archimedean property.
theorem exists_nat_half_bound(x: Real) {
    x.is_positive implies exists(n: Nat) {
        x < Real.from_rat(Rat.from_nat(n)) / two
    }
} by {
    if x.is_positive {
        // Find N such that 2x < N
        let two_x = two * x
        Real.1.is_positive
        two.is_positive
        x.is_positive
        two_x.is_positive

        let n: Nat satisfy {
            Real.from_rat(Rat.from_nat(n)) > two_x
        }

        Real.from_rat(Rat.from_nat(n)) > two * x
        two != Real.0
        two.is_positive

        // Use div_lt_div_pos: two * x < Real.from_rat(Rat.from_nat(n))
        // implies (two * x) / two < Real.from_rat(Rat.from_nat(n)) / two
        two * x < Real.from_rat(Rat.from_nat(n))
        (two * x) / two < Real.from_rat(Rat.from_nat(n)) / two

        // Simplify (two * x) / two = x
        two * (x / two) = x
        (two * x) / two = x

        x < Real.from_rat(Rat.from_nat(n)) / two
    }
}

/// Simplification lemma: (a / b) / b = a / (b * b) when b != 0.
theorem div_div_same(a: Real, b: Real) {
    b != Real.0 implies (a / b) / b = a / (b * b)
} by {
    if b != Real.0 {
        (a / b) / b = (a * b.inverse) * b.inverse
        (a * b.inverse) * b.inverse = a * (b.inverse * b.inverse)

        // Use inverse_dist from field.ac: (b * b).inverse = b.inverse * b.inverse
        // Since inverse is the same as inverse for reals
        b * b != Real.0
        (b * b).inverse = b.inverse * b.inverse

        a * (b.inverse * b.inverse) = a * (b * b).inverse
        a * (b * b).inverse = a / (b * b)
    }
}

/// If x < n/2, then x/n < 1/2 (when n > 0).
/// This follows from div_mul_cancel_right but currently doesn't verify due to
/// the prover not finding the right instantiation automatically.
theorem div_preserves_half_bound(x: Real, n: Nat) {
    x < Real.from_rat(Rat.from_nat(n)) / two
    and n >= Nat.1
    implies x / Real.from_rat(Rat.from_nat(n)) < Real.1 / two
} by {
    let r_n = Real.from_rat(Rat.from_nat(n))
    Nat.1 <= n
    Rat.1 <= Rat.from_nat(n)
    Real.1 <= r_n
    r_n > Real.0
    r_n != Real.0
    two != Real.0
    two.is_positive

    // x < r_n / 2, so divide both sides by r_n
    x < r_n / two
    // Use div_lt_div_pos
    x / r_n < (r_n / two) / r_n

    // Use div_div_same: (r_n / two) / r_n = r_n / (two * r_n)
    r_n * two.inverse = r_n / two
    r_n * two = two * r_n
    r_n * two / r_n = two or Real.0 = r_n
    r_n * two != Real.0 or Real.0 = two or Real.0 = r_n
    (r_n / two) / r_n = r_n / (two * r_n)

    // Use div_mul_cancel_right: a / (b * a) = 1 / b
    // With a = r_n, b = two: r_n / (two * r_n) = 1 / two
    r_n / (two * r_n) = Real.1 / two

    x / r_n < Real.1 / two
    x / Real.from_rat(Rat.from_nat(n)) < Real.1 / two
}

/// Helper predicate: sequence is always nonzero
define always_nonzero(f: Nat -> Real) -> Bool {
    forall(k: Nat) {
        f(k) != Real.0
    }
}

/// exp_term(x, n) is nonzero when x is nonzero.
theorem exp_term_nonzero(x: Real, n: Nat) {
    x != Real.0 implies exp_term(x, n) != Real.0
} by {
    if x != Real.0 {
        // exp_term(x, n) = x^n / n!
        // x != 0 implies x^n != 0
        // n! > 0 always, so n! != 0
        // Therefore x^n / n! != 0
        x.pow(n) != Real.0
        Real.from_rat(Rat.from_nat(n.factorial)) != Real.0
        exp_term(x, n) != Real.0
    }
}

/// When x != 0, exp_term(x) is always nonzero.
theorem exp_term_always_nonzero(x: Real) {
    x != Real.0 implies always_nonzero(exp_term(x))
} by {
    if x != Real.0 {
        forall(k: Nat) {
            exp_term(x, k) != Real.0
        }
        always_nonzero(exp_term(x))
    }
}

/// A sequence has a ratio bound if the ratio of consecutive terms is bounded by alpha.
/// This is used for ratio test-style convergence arguments.
define has_ratio_bound(f: Nat -> Real, alpha: Real) -> Bool {
    forall(k: Nat) {
        f(k) != Real.0 implies (f(k.suc).abs / f(k).abs) < alpha
    }
}

/// A sequence eventually has a ratio bound if there exists an index after which
/// the ratio of consecutive terms is bounded by alpha.
define has_ratio_bound_eventually(f: Nat -> Real, alpha: Real) -> Bool {
    exists(big_n: Nat) {
        forall(k: Nat) {
            k >= big_n and f(k) != Real.0 implies (f(k.suc).abs / f(k).abs) < alpha
        }
    }
}

/// If a sequence has ratio bound alpha and is always nonzero,
/// then |f(k)| ≤ |f(0)| * alpha^k.
/// This is proven by induction on k.
theorem ratio_bound_geometric_bound(f: Nat -> Real, alpha: Real, k: Nat) {
    has_ratio_bound(f, alpha) and Real.0 <= alpha and always_nonzero(f)
    implies
    f(k).abs <= f(Nat.0).abs * alpha.pow(k)
} by {
    // Proof by induction on k
    define p(n: Nat) -> Bool {
        has_ratio_bound(f, alpha) and Real.0 <= alpha and always_nonzero(f)
        implies
        f(n).abs <= f(Nat.0).abs * alpha.pow(n)
    }

    // Base case: k = 0
    // f(0).abs ≤ f(0).abs * alpha^0 = f(0).abs * 1 = f(0).abs
    alpha.pow(Nat.0) = Real.1
    f(Nat.0).abs * alpha.pow(Nat.0) = f(Nat.0).abs
    p(Nat.0)

    // Inductive step
    forall(n: Nat) {
        if p(n) {
            if has_ratio_bound(f, alpha) and Real.0 <= alpha and always_nonzero(f) {
                // By IH: f(n).abs ≤ f(0).abs * alpha^n
                f(n).abs <= f(Nat.0).abs * alpha.pow(n)

                // By always_nonzero: f(n) != 0
                f(n) != Real.0

                // By has_ratio_bound: |f(n+1)| / |f(n)| < alpha
                (f(n.suc).abs / f(n).abs) < alpha

                // Since f(n) != 0, we have |f(n)| > 0
                f(n).abs >= Real.0
                f(n).abs > Real.0
                f(n).abs != Real.0

                // From: |f(n+1)| / |f(n)| < alpha and |f(n)| > 0
                // We get: |f(n+1)| < |f(n)| * alpha
                // (using div_lt_mul_pos from real_field.ac)
                f(n.suc).abs < f(n).abs * alpha

                // By IH: f(n).abs ≤ f(0).abs * alpha^n
                // So: f(n).abs * alpha ≤ f(0).abs * alpha^n * alpha = f(0).abs * alpha^(n+1)
                alpha.pow(n.suc) = alpha * alpha.pow(n)
                f(n).abs * alpha <= f(Nat.0).abs * alpha.pow(n) * alpha
                f(Nat.0).abs * alpha.pow(n) * alpha = f(Nat.0).abs * alpha.pow(n.suc)

                // Combining: f(n+1).abs < f(n).abs * alpha ≤ f(0).abs * alpha^(n+1)
                f(n.suc).abs <= f(Nat.0).abs * alpha.pow(n.suc)
            }
            p(n.suc)
        }
    }

    // By induction, p holds for all n
    p(k)
}

/// Ratio test for absolute convergence.
/// If a series has ratio bound alpha < 1 and is always nonzero, then it converges absolutely.
///
/// Proof strategy:
/// 1. Use ratio_bound_geometric_bound to show |f(k)| ≤ |f(0)| * alpha^k
/// 2. The series is dominated by the geometric series |f(0)| * alpha^k
/// 3. Since alpha < 1, the geometric series converges (by geom_converges)
/// 4. Apply absolutely_converges_comparison to conclude
theorem ratio_bound_implies_abs_conv(f: Nat -> Real, alpha: Real) {
    has_ratio_bound(f, alpha) and alpha < Real.1 and Real.0 <= alpha and always_nonzero(f)
    implies
    absolutely_converges(f)
} by {
    if has_ratio_bound(f, alpha) and alpha < Real.1 and Real.0 <= alpha and always_nonzero(f) {
        // By ratio_bound_geometric_bound, for all k: |f(k)| ≤ |f(0)| * alpha^k
        forall(k: Nat) {
            f(k).abs <= f(Nat.0).abs * alpha.pow(k)
        }

        // Define comparison sequence: b(k) = |f(0)| * alpha^k
        let c = f(Nat.0).abs
        define b(k: Nat) -> Real {
            c * alpha.pow(k)
        }

        // Show that abs_fn(f) ≤ b pointwise
        forall(k: Nat) {
            abs_fn(f)(k) = f(k).abs
            f(k).abs <= c * alpha.pow(k)
            abs_fn(f)(k) <= b(k)
        }
        seq_lte(abs_fn(f), b)

        // Show b is nonnegative
        forall(k: Nat) {
            Real.0 <= alpha
            Real.0 <= alpha.pow(k)
            Real.0 <= c * alpha.pow(k)
            Real.0 <= b(k)
        }
        is_lower_bound(b, Real.0)

        // Show that partial(b) converges
        // b(k) = c * alpha^k, so partial(b) = c * partial(alpha.pow)
        // Since alpha < 1 and alpha >= 0, we have alpha.abs = alpha < 1
        // So by geom_converges, partial(alpha.pow) converges
        Real.0 <= alpha
        not alpha.is_negative
        alpha.abs = alpha
        alpha.abs < Real.1
        converges(partial(alpha.pow))
        // partial(mul_seq(c, alpha.pow)) = mul_seq(c, partial(alpha.pow))
        forall(k: Nat) {
            c * alpha.pow(k) = b(k)
            c * alpha.pow(k) = mul_seq(c, alpha.pow, k)
            b(k) = mul_seq(c, alpha.pow, k)
        }
        b = mul_seq(c, alpha.pow)
        partial(b) = partial(mul_seq(c, alpha.pow))
        partial(mul_seq(c, alpha.pow)) = mul_seq(c, partial(alpha.pow))
        converges(mul_seq(c, partial(alpha.pow)))
        converges(partial(b))

        // Apply comparison test
        absolutely_converges(f)
    }
}

/// Eventual ratio test for absolute convergence.
/// If a series eventually has ratio bound alpha < 1 and is always nonzero, then it converges absolutely.
///
/// Proof strategy:
/// 1. Find big_n such that for all k >= big_n, |f(k+1)| / |f(k)| < alpha
/// 2. Show that tail(f, big_n) has_ratio_bound alpha and is always nonzero
/// 3. Apply ratio_bound_implies_abs_conv to the tail
/// 4. Use partial_tail_conv_imp_partial_conv to conclude the full series converges
theorem eventual_ratio_bound_implies_abs_conv(f: Nat -> Real, alpha: Real) {
    has_ratio_bound_eventually(f, alpha) and alpha < Real.1 and Real.0 <= alpha and always_nonzero(f)
    implies
    absolutely_converges(f)
} by {
    if has_ratio_bound_eventually(f, alpha) and alpha < Real.1 and Real.0 <= alpha and always_nonzero(f) {
        // Get the index after which the ratio bound holds
        let big_n: Nat satisfy {
            forall(k: Nat) {
                k >= big_n and f(k) != Real.0 implies (f(k.suc).abs / f(k).abs) < alpha
            }
        }

        // Show that tail(f, big_n) is always nonzero
        forall(i: Nat) {
            tail(f, big_n)(i) = f(big_n + i)
            f(big_n + i) != Real.0
            tail(f, big_n)(i) != Real.0
        }
        always_nonzero(tail(f, big_n))

        // Show that tail(f, big_n) has ratio bound alpha
        // For any index i in the tail: tail(f, big_n)(i) = f(big_n + i)
        // We have big_n + i >= big_n, so the ratio bound applies
        forall(i: Nat) {
            if tail(f, big_n)(i) != Real.0 {
                f(big_n + i) != Real.0
                big_n + i >= big_n
                (f(big_n + i.suc).abs / f(big_n + i).abs) < alpha
                (tail(f, big_n)(i.suc).abs / tail(f, big_n)(i).abs) < alpha
            }
        }
        exists(k0: Nat) {
            tail(f, big_n)(k0) != Real.0 and
            not (tail(f, big_n)(k0.suc).abs / tail(f, big_n)(k0).abs) < alpha
        } or has_ratio_bound(tail(f, big_n), alpha)
        has_ratio_bound(tail(f, big_n), alpha)

        // Apply ratio test to the tail
        absolutely_converges(tail(f, big_n))

        // The tail converges absolutely, so its partial sums converge
        converges(partial(abs_fn(tail(f, big_n))))

        // By partial_tail_conv_imp_partial_conv, the full series converges
        converges(partial(abs_fn(f)))
        absolutely_converges(f)
    }
}

/// For all x, the series with terms exp_term(x) converges absolutely.
///
/// Proof strategy:
/// 1. For x = 0: series is eventually zero, hence converges
/// 2. For x ≠ 0: use eventual ratio test with ratio bound 1/2
///    a. Show exp_term(x) is always nonzero
///    b. Find N such that |x| < N/2 (Archimedean property)
///    c. For n >= N, show |x|/(n+1) < 1/2 (by exp_term_ratio)
///    d. Apply eventual_ratio_bound_implies_abs_conv
theorem exp_term_abs_converges(x: Real) {
    absolutely_converges(exp_term(x))
} by {
    if x = Real.0 {
        // For x = 0, all terms are zero for n >= 1, so the series trivially converges
        forall(i: Nat) {
            tail(abs_fn(exp_term(Real.0)), Nat.1)(i) = abs_fn(exp_term(Real.0))(Nat.1 + i)
            abs_fn(exp_term(Real.0))(Nat.1 + i) = exp_term(Real.0, Nat.1 + i).abs
            Nat.1 + i >= Nat.1
            exp_term(Real.0, Nat.1 + i) = Real.0
            exp_term(Real.0, Nat.1 + i).abs = Real.0
            tail(abs_fn(exp_term(Real.0)), Nat.1)(i) = Real.0
            constant[Nat, Real](Real.0, i) = Real.0
            tail(abs_fn(exp_term(Real.0)), Nat.1)(i) = constant[Nat, Real](Real.0, i)
        }
        tail(abs_fn(exp_term(Real.0)), Nat.1) = constant[Nat, Real](Real.0)
        partial(constant[Nat, Real](Real.0)) = constant[Nat, Real](Real.0)
        converges(constant[Nat, Real](Real.0))
        converges(partial(tail(abs_fn(exp_term(Real.0)), Nat.1)))
        converges(partial(abs_fn(exp_term(Real.0))))
        absolutely_converges(exp_term(Real.0))
    } else {
        // For x != 0, show exp_term(x) converges absolutely using the ratio test
        x != Real.0

        // Show exp_term(x) is always nonzero
        always_nonzero(exp_term(x))

        // Show exp_term(x) has eventual ratio bound 1/2
        x.abs.is_positive

        // By Archimedean property: there exists N such that |x| * 2 < N
        let two_x = x.abs * two
        two.is_positive
        two_x.is_positive
        exists(big_n: Nat) {
            x.abs * two < Real.from_rat(Rat.from_nat(big_n))
        }
        let big_n: Nat satisfy {
            x.abs * two < Real.from_rat(Rat.from_nat(big_n))
        }

        // From x.abs * 2 < big_n, we get x.abs < big_n/2
        two > Real.0
        two != Real.0
        x.abs < Real.from_rat(Rat.from_nat(big_n)) / two

        define p(n_bound: Nat) -> Bool {
            forall(k: Nat) {
                k >= n_bound and exp_term(x, k) != Real.0 implies (exp_term(x, k.suc).abs / exp_term(x, k).abs) < Real.1 / two
            }
        }

        // Show that for all k >= big_n, the ratio < 1/2
        forall(k: Nat) {
            if k >= big_n and exp_term(x, k) != Real.0 {
                // By exp_term_ratio: |exp_term(x, k+1)| / |exp_term(x, k)| = |x| / (k+1)
                exp_term(x, k).abs != Real.0
                (exp_term(x, k.suc).abs / exp_term(x, k).abs) = x.abs / Real.from_rat(Rat.from_nat(k.suc))

                // Since k >= big_n, we have: k.suc > big_n
                // From x.abs < big_n/2 and big_n < k.suc, we get x.abs < k.suc/2
                // Therefore x.abs / k.suc < (k.suc / 2) / k.suc = 1/2
                let ksucc = Real.from_rat(Rat.from_nat(k.suc))
                k.suc > k
                k >= big_n
                k.suc > big_n
                big_n < k.suc
                Rat.from_nat(big_n) < Rat.from_nat(k.suc)
                Real.from_rat(Rat.from_nat(big_n)) < Real.from_rat(Rat.from_nat(k.suc))
                Real.from_rat(Rat.from_nat(big_n)) < ksucc
                ksucc > Real.0
                ksucc != Real.0
                ksucc / two > Real.from_rat(Rat.from_nat(big_n)) / two
                x.abs < ksucc / two

                // Use our new field lemma: x.abs / ksucc < 1/2 from x.abs < ksucc/2
                x.abs / ksucc < (ksucc / two) / ksucc
                ksucc / (ksucc * two) = Real.1 / two
                (ksucc / two) / ksucc = Real.1 / two
                x.abs / ksucc < Real.1 / two

                (exp_term(x, k.suc).abs / exp_term(x, k).abs) < Real.1 / two
            }
        }

        // So exp_term(x) has eventual ratio bound 1/2
        exists(k0: Nat) {
            k0 >= big_n and
            exp_term(x, k0) != Real.0 and
            not (exp_term(x, k0.suc).abs / exp_term(x, k0).abs) < Real.1 / two
        } or p(big_n)
        p(big_n)

        exists(n_bound: Nat) {
            n_bound = big_n and p(n_bound)
        }

        exists(n_bound: Nat) { p(n_bound) }
        not exists(n_bound: Nat) {
            forall(k: Nat) {
                k >= n_bound and exp_term(x, k) != Real.0 implies (exp_term(x, k.suc).abs / exp_term(x, k).abs) < Real.1 / two
            }
        } or has_ratio_bound_eventually(exp_term(x), Real.1 / two)
        has_ratio_bound_eventually(exp_term(x), Real.1 / two)

        // Apply the eventual ratio test
        // Check preconditions: has_ratio_bound_eventually, alpha < 1, alpha >= 0, always_nonzero
        // Need to show 1/2 < 1
        two > Real.1
        two.is_positive
        not two.is_negative
        two.inverse < Real.1.inverse
        Real.1.inverse = Real.1
        two.inverse < Real.1
        Real.1 / two = Real.1 * two.inverse
        Real.1 * two.inverse < Real.1 * Real.1
        Real.1 * Real.1 = Real.1
        Real.1 / two < Real.1
        Real.0 <= Real.1 / two
        absolutely_converges(exp_term(x))
    }
}

/// The exponential function x.exp = e^x.
/// Defined as the limit of the partial sums of the Taylor series.
attributes Real {
    /// The exponential function x.exp = e^x.
    /// Defined as the limit of the partial sums of the Taylor series.
    define exp(self) -> Real {
        limit(partial(exp_term(self)))
    }
}

/// The partial sums of exp_term converge.
theorem exp_term_partial_converges(x: Real) {
    converges(partial(exp_term(x)))
} by {
    // Since exp_term(x) converges absolutely, its partial sums converge
    absolutely_converges(exp_term(x))
    converges(partial(abs_fn(exp_term(x))))

    // Need to show converges(partial(exp_term(x)))
    // This follows from absolute convergence implying convergence
    converges(partial(exp_term(x)))
}

// Binomial coefficient lemmas for Real.exp

/// Helper: Rat.from_nat preserves multiplication.
theorem rat_from_nat_mul(m: Nat, n: Nat) {
    Rat.from_nat(m * n) = Rat.from_nat(m) * Rat.from_nat(n)
} by {
    Rat.from_nat(m * n) = Rat.from_int(Int.from_nat(m * n))
    Int.from_nat(m * n) = Int.from_nat(m) * Int.from_nat(n)
    Rat.from_int(Int.from_nat(m) * Int.from_nat(n)) =
        Rat.from_int(Int.from_nat(m)) * Rat.from_int(Int.from_nat(n))
}

/// Helper: Real.from_rat preserves multiplication (imported from real_ring).
/// This is mul_from_rat from real_ring.ac

/// Helper: relating choose coefficient to factorials.
theorem choose_factorial_nat(n: Nat, k: Nat) {
    k <= n implies
    n.binom(k) * k.factorial * (n - k).factorial = n.factorial
}

/// Helper: Rat.from_nat distributes over products of three factors.
theorem rat_from_nat_mul3(a: Nat, b: Nat, c: Nat) {
    Rat.from_nat(a * b * c) = Rat.from_nat(a) * Rat.from_nat(b) * Rat.from_nat(c)
} by {
    Rat.from_nat(a * b * c) = Rat.from_nat(a * b) * Rat.from_nat(c)
    Rat.from_nat(a * b) = Rat.from_nat(a) * Rat.from_nat(b)
}

/// Helper: Real.from_rat distributes over products of three rationals.
theorem real_from_rat_mul3(a: Rat, b: Rat, c: Rat) {
    Real.from_rat(a * b * c) = Real.from_rat(a) * Real.from_rat(b) * Real.from_rat(c)
} by {
    Real.from_rat(a * b * c) = Real.from_rat(a * b) * Real.from_rat(c)
    Real.from_rat(a * b) = Real.from_rat(a) * Real.from_rat(b)
}

/// Helper: relating choose coefficient to factorials via reals.
theorem choose_factorial_real(n: Nat, k: Nat) {
    k <= n implies
    from_nat[Real](n.binom(k)) * Real.from_rat(Rat.from_nat(k.factorial)) * Real.from_rat(Rat.from_nat((n - k).factorial)) =
        Real.from_rat(Rat.from_nat(n.factorial))
} by {
    if k <= n {
        // Use choose_factorial_nat
        n.binom(k) * k.factorial * (n - k).factorial = n.factorial

        // Apply Rat.from_nat to both sides
        Rat.from_nat(n.binom(k) * k.factorial * (n - k).factorial) = Rat.from_nat(n.factorial)

        // Use rat_from_nat_mul3
        Rat.from_nat(n.binom(k)) * Rat.from_nat(k.factorial) * Rat.from_nat((n - k).factorial) =
            Rat.from_nat(n.factorial)

        // Apply Real.from_rat to both sides
        Real.from_rat(Rat.from_nat(n.binom(k)) * Rat.from_nat(k.factorial) * Rat.from_nat((n - k).factorial)) =
            Real.from_rat(Rat.from_nat(n.factorial))

        // Use real_from_rat_mul3
        Real.from_rat(Rat.from_nat(n.binom(k))) * Real.from_rat(Rat.from_nat(k.factorial)) * Real.from_rat(Rat.from_nat((n - k).factorial)) =
            Real.from_rat(Rat.from_nat(n.factorial))

        // Use from_nat_is_from_rat for n.binom(k)
        from_nat[Real](n.binom(k)) = Real.from_rat(Rat.from_nat(n.binom(k)))

        // Add conversion facts
        Real.from_rat(Rat.from_nat(k.factorial)) = from_nat[Real](k.factorial)
        Real.from_rat(Rat.from_nat((n - k).factorial)) = from_nat[Real]((n - k).factorial)
        Real.from_rat(Rat.from_nat(n.factorial)) = from_nat[Real](n.factorial)
        from_nat[Real](n.binom(k) * k.factorial) = from_nat[Real](n.binom(k)) * from_nat[Real](k.factorial)
        from_nat[Real](n.binom(k) * k.factorial) * from_nat[Real]((n - k).factorial) = from_nat[Real](n.binom(k) * k.factorial * (n - k).factorial)

        from_nat[Real](n.binom(k)) * Real.from_rat(Rat.from_nat(k.factorial)) * Real.from_rat(Rat.from_nat((n - k).factorial)) =
            Real.from_rat(Rat.from_nat(n.factorial))
    }
}

/// Helper: inverse relation from binomial coefficient identity.
/// This states that 1/(k! * (n-k)!) = C(n,k)/n!
theorem choose_factorial_inverse(n: Nat, k: Nat) {
    k <= n implies Real.1 / (Real.from_rat(Rat.from_nat(k.factorial)) * Real.from_rat(Rat.from_nat((n - k).factorial))) = from_nat[Real](n.binom(k)) / Real.from_rat(Rat.from_nat(n.factorial))
} by {
    if k <= n {
        // Both factorials are non-zero
        Real.from_rat(Rat.from_nat(k.factorial)) != Real.0
        Real.from_rat(Rat.from_nat((n - k).factorial)) != Real.0
        Real.from_rat(Rat.from_nat(n.factorial)) != Real.0

        // From choose_factorial_real:
        // C(n,k) * k! * (n-k)! = n!
        from_nat[Real](n.binom(k)) *
        Real.from_rat(Rat.from_nat(k.factorial)) *
        Real.from_rat(Rat.from_nat((n - k).factorial)) =
            Real.from_rat(Rat.from_nat(n.factorial))

        // The product k! * (n-k)! is non-zero
        Real.from_rat(Rat.from_nat(k.factorial)) * Real.from_rat(Rat.from_nat((n - k).factorial)) != Real.0

        // Divide both sides by (k! * (n-k)!)
        from_nat[Real](n.binom(k)) =
            Real.from_rat(Rat.from_nat(n.factorial)) /
            (Real.from_rat(Rat.from_nat(k.factorial)) * Real.from_rat(Rat.from_nat((n - k).factorial)))

        // Divide both sides by n!
        from_nat[Real](n.binom(k)) / Real.from_rat(Rat.from_nat(n.factorial)) =
            Real.1 /
            (Real.from_rat(Rat.from_nat(k.factorial)) * Real.from_rat(Rat.from_nat((n - k).factorial)))
    }
}

/// Micro-lemma: a * (1 / b) = a / b
theorem mul_one_over(a: Real, b: Real) {
    b != Real.0 implies a * (Real.1 / b) = a / b
}

/// Micro-lemma: a * (b / c) = (a * b) / c
theorem mul_frac_assoc(a: Real, b: Real, c: Real) {
    c != Real.0 implies a * (b / c) = (a * b) / c
}

/// Micro-lemma: (a * b) * c = a * (b * c)
theorem mul_assoc_real(a: Real, b: Real, c: Real) {
    (a * b) * c = a * (b * c)
}

/// Helper lemma step 1: Product of exp_terms gives combined fraction (VERIFIED).
theorem exp_term_product_combined(x: Real, y: Real, n: Nat, k: Nat) {
    k <= n implies
    exp_term(x, k) * exp_term(y, n - k) =
        (x.pow(k) * y.pow(n - k)) /
        (Real.from_rat(Rat.from_nat(k.factorial)) * Real.from_rat(Rat.from_nat((n - k).factorial)))
} by {
    if k <= n {
        Real.from_rat(Rat.from_nat(k.factorial)) != Real.0
        Real.from_rat(Rat.from_nat((n - k).factorial)) != Real.0

        exp_term(x, k) = x.pow(k) / Real.from_rat(Rat.from_nat(k.factorial))
        exp_term(y, n - k) = y.pow(n - k) / Real.from_rat(Rat.from_nat((n - k).factorial))

        // Apply div_mul_div: (a/b) * (c/d) = (a*c) / (b*d)
        (x.pow(k) / Real.from_rat(Rat.from_nat(k.factorial))) *
        (y.pow(n - k) / Real.from_rat(Rat.from_nat((n - k).factorial))) =
            (x.pow(k) * y.pow(n - k)) /
            (Real.from_rat(Rat.from_nat(k.factorial)) * Real.from_rat(Rat.from_nat((n - k).factorial)))
    }
}

/// Substitution lemma: substitute inverse identity into a product
theorem substitute_inverse_identity(a: Real, b: Real, c: Real) {
    b != Real.0 and c != Real.0 and Real.1 / b = c implies a * (Real.1 / b) = a * c
}

/// Intermediate step: numerator transformation using inverse identity
theorem numerator_transform_step(x: Real, y: Real, n: Nat, k: Nat) {
    k <= n and Real.from_rat(Rat.from_nat(n.factorial)) != Real.0 implies
    (x.pow(k) * y.pow(n - k)) * (from_nat[Real](n.binom(k)) / Real.from_rat(Rat.from_nat(n.factorial))) =
        (from_nat[Real](n.binom(k)) * x.pow(k) * y.pow(n - k)) / Real.from_rat(Rat.from_nat(n.factorial))
} by {
    if k <= n and Real.from_rat(Rat.from_nat(n.factorial)) != Real.0 {
        (x.pow(k) * y.pow(n - k)) * (from_nat[Real](n.binom(k)) / Real.from_rat(Rat.from_nat(n.factorial))) =
            ((x.pow(k) * y.pow(n - k)) * from_nat[Real](n.binom(k))) / Real.from_rat(Rat.from_nat(n.factorial))
        (x.pow(k) * y.pow(n - k)) * from_nat[Real](n.binom(k)) = from_nat[Real](n.binom(k)) * x.pow(k) * y.pow(n - k)
    }
}

/// Helper lemma step 2: Use binomial coefficient identity to transform denominator.
theorem binomial_fraction_transform(x: Real, y: Real, n: Nat, k: Nat) {
    k <= n implies
    (x.pow(k) * y.pow(n - k)) /
    (Real.from_rat(Rat.from_nat(k.factorial)) * Real.from_rat(Rat.from_nat((n - k).factorial))) =
        (from_nat[Real](n.binom(k)) * x.pow(k) * y.pow(n - k)) / Real.from_rat(Rat.from_nat(n.factorial))
} by {
    Real.from_rat(Rat.from_nat(k.factorial)) != Real.0
    Real.from_rat(Rat.from_nat((n - k).factorial)) != Real.0
    Real.from_rat(Rat.from_nat(n.factorial)) != Real.0
    Real.from_rat(Rat.from_nat(k.factorial)) * Real.from_rat(Rat.from_nat((n - k).factorial)) != Real.0
    k <= n implies Real.1 / (Real.from_rat(Rat.from_nat(k.factorial)) * Real.from_rat(Rat.from_nat((n - k).factorial))) = from_nat[Real](n.binom(k)) / Real.from_rat(Rat.from_nat(n.factorial))

    let a = x.pow(k) * y.pow(n - k)
    let b = Real.from_rat(Rat.from_nat(k.factorial))
    let c = Real.from_rat(Rat.from_nat((n - k).factorial))
    let d = from_nat[Real](n.binom(k))
    let f = Real.from_rat(Rat.from_nat(n.factorial))

    // Division definitions
    a * (b * c).inverse = a / (b * c)
    d * a * f.inverse = (d * a) / f

    // Commutativity and associativity
    d * a = a * d
    a * (d * f.inverse) = a * d * f.inverse
    Real.1 * (b * c).inverse = (b * c).inverse

    a / (b * c) = (d * a) / f
}

/// Key lemma: the product of exp_term(x, k) and exp_term(y, n-k) equals
/// the binomial coefficient form divided by n!.
/// This shows that (x^k/k!) * (y^(n-k)/(n-k)!) = (n.binom(k) * x^k * y^(n-k)) / n!
theorem exp_term_product_binomial(x: Real, y: Real, n: Nat, k: Nat) {
    k <= n implies
    exp_term(x, k) * exp_term(y, n - k) =
        (from_nat[Real](n.binom(k)) * x.pow(k) * y.pow(n - k)) / Real.from_rat(Rat.from_nat(n.factorial))
} by {
    // Use exp_term_product_combined
    exp_term(x, k) * exp_term(y, n - k) =
        (x.pow(k) * y.pow(n - k)) /
        (Real.from_rat(Rat.from_nat(k.factorial)) * Real.from_rat(Rat.from_nat((n - k).factorial)))

    // Use binomial_fraction_transform
    (x.pow(k) * y.pow(n - k)) /
    (Real.from_rat(Rat.from_nat(k.factorial)) * Real.from_rat(Rat.from_nat((n - k).factorial))) =
        (from_nat[Real](n.binom(k)) * x.pow(k) * y.pow(n - k)) / Real.from_rat(Rat.from_nat(n.factorial))
}

/// Two functions are equal if they're equal pointwise on a domain
theorem function_extensionality_on_range(f: Nat -> Real, g: Nat -> Real, n: Nat) {
    (forall(k: Nat) { k < n implies f(k) = g(k) })
    implies
    partial(f, n) = partial(g, n)
} by {
    from list import partial_pointwise_eq
}

/// div_fn equals mul_fn with inverse
theorem div_fn_as_mul_fn(f: Nat -> Real, c: Real) {
    c != Real.0 implies
    div_fn(f, c) = mul_fn(c.inverse, f)
} by {
    if c != Real.0 {
        // Show pointwise equality
        forall(k: Nat) {
            div_fn(f, c)(k) = f(k) / c
            f(k) / c = f(k) * c.inverse
            mul_fn(c.inverse, f)(k) = c.inverse * f(k)
            c.inverse * f(k) = f(k) * c.inverse
            div_fn(f, c)(k) = mul_fn(c.inverse, f)(k)
        }

        // Use function extensionality (if we had it globally, but we only have it on ranges)
        // For now, rely on the prover to infer function equality from pointwise equality
    }
}

/// Factoring a constant divisor out of a partial sum.
/// sum(div_fn(f, c)) = (sum f) / c when c != 0
theorem partial_div_fn(f: Nat -> Real, c: Real, n: Nat) {
    c != Real.0 implies
    partial(div_fn(f, c), n) = partial(f, n) / c
} by {
    // Show div_fn(f, c) = mul_fn(c.inverse, f)
    div_fn(f, c) = mul_fn(c.inverse, f)

    // Apply partial_scalar_mul
    c.inverse * partial(f, n) = partial(mul_fn(c.inverse, f), n)

    // Substitute
    partial(div_fn(f, c), n) = partial(mul_fn(c.inverse, f), n)
    partial(mul_fn(c.inverse, f), n) = c.inverse * partial(f, n)

    // Convert multiplication to division
    c.inverse * partial(f, n) = partial(f, n) * c.inverse
    partial(f, n) * c.inverse = partial(f, n) / c
    partial(div_fn(f, c), n) = partial(f, n) / c
}

/// The Cauchy product of exp_term sequences equals exp_term of the sum.
theorem exp_cauchy_product_eq(x: Real, y: Real, n: Nat) {
    cauchy_product(exp_term(x), exp_term(y), n) = exp_term(x + y, n)
} by {
    let n_fact = Real.from_rat(Rat.from_nat(n.factorial))
    n_fact != Real.0

    // Apply exp_term_product_binomial to show pointwise equality
    forall(k: Nat) {
        if k <= n {
            k <= n implies exp_term(x, k) * exp_term(y, n - k) = (from_nat[Real](n.binom(k)) * x.pow(k) * y.pow(n - k)) / Real.from_rat(Rat.from_nat(n.factorial))
            exp_term(x, k) * exp_term(y, n - k) = binomial_term[Real](x, y, n, k) / n_fact
            exp_term(x, k) * exp_term(y, n - k) = div_fn(binomial_term[Real](x, y, n), n_fact)(k)
        }
    }

    // Show the functions are equal on the range [0, n]
    define f(k: Nat) -> Real {
        exp_term(x, k) * exp_term(y, n - k)
    }
    define g(k: Nat) -> Real {
        div_fn(binomial_term[Real](x, y, n), n_fact)(k)
    }

    forall(k: Nat) {
        if k < n.suc {
            k <= n
            f(k) = g(k)
        }
    }

    // Use function_extensionality_on_range
    partial(f, n.suc) = partial(g, n.suc)
    partial(g, n.suc) = partial(div_fn(binomial_term[Real](x, y, n), n_fact), n.suc)

    // Use partial_div_fn to factor out the division
    partial(div_fn(binomial_term[Real](x, y, n), n_fact), n.suc) = partial(binomial_term[Real](x, y, n), n.suc) / n_fact

    // Apply binomial theorem
    (x + y).pow(n) = partial(binomial_term[Real](x, y, n), n.suc)
    partial(binomial_term[Real](x, y, n), n.suc) / n_fact = (x + y).pow(n) / n_fact

    // This is exp_term(x+y, n)
    exp_term(x + y, n) = (x + y).pow(n) / n_fact

    // Connect through the chain of equalities:
    // partial(f, n.suc) = partial(g, n.suc) by function_extensionality
    // partial(g, n.suc) = partial(div_fn(binomial_term, n_fact), n.suc) by def of g
    // partial(div_fn(binomial_term, n_fact), n.suc) = partial(binomial_term, n.suc) / n_fact by partial_div_fn
    // partial(binomial_term, n.suc) = (x+y)^n by binomial theorem
    // Therefore partial(f, n.suc) = (x+y)^n / n_fact

    // We've already shown these steps above, so:
    partial(f, n.suc) = partial(g, n.suc)
    partial(g, n.suc) = partial(div_fn(binomial_term[Real](x, y, n), n_fact), n.suc)
    partial(div_fn(binomial_term[Real](x, y, n), n_fact), n.suc) = partial(binomial_term[Real](x, y, n), n.suc) / n_fact
    partial(binomial_term[Real](x, y, n), n.suc) = (x + y).pow(n)

    // Chain them together
    partial(f, n.suc) = (x + y).pow(n) / n_fact

    // Connect cauchy_product to partial(f, n.suc) using function extensionality
    // cauchy_coefficient(exp_term(x), exp_term(y), n)(k) = exp_term(x, k) * exp_term(y, n - k)
    // which equals f(k) by definition, so the functions are equal
    define cc(k: Nat) -> Real {
        cauchy_coefficient(exp_term(x), exp_term(y), n)(k)
    }

    forall(k: Nat) {
        if k < n.suc {
            // cauchy_coefficient unfolds to the product
            cc(k) = cauchy_coefficient(exp_term(x), exp_term(y), n)(k)
            cc(k) = exp_term(x, k) * exp_term(y, n - k)
            cc(k) = f(k)
        }
    }

    // Use function extensionality
    partial(cc, n.suc) = partial(f, n.suc)

    // By definition: cauchy_product(a, b, n) = sum(map(n.suc.range, cauchy_coefficient(a, b, n)))
    // which equals partial(cauchy_coefficient(a, b, n), n.suc)
    cauchy_product(exp_term(x), exp_term(y), n) = partial(cauchy_coefficient(exp_term(x), exp_term(y), n), n.suc)
    partial(cauchy_coefficient(exp_term(x), exp_term(y), n), n.suc) = partial(cc, n.suc)

    // Therefore
    cauchy_product(exp_term(x), exp_term(y), n) = partial(f, n.suc)
    partial(f, n.suc) = (x + y).pow(n) / n_fact
    cauchy_product(exp_term(x), exp_term(y), n) = (x + y).pow(n) / n_fact
    exp_term(x + y, n) = (x + y).pow(n) / n_fact
}

/// The Cauchy product sequence equals the exp_term sequence for the sum.
theorem exp_cauchy_seq_eq(x: Real, y: Real) {
    cauchy_seq(exp_term(x), exp_term(y)) = exp_term(x + y)
} by {
    forall(n: Nat) {
        cauchy_seq(exp_term(x), exp_term(y))(n) = cauchy_product(exp_term(x), exp_term(y), n)
        cauchy_product(exp_term(x), exp_term(y), n) = exp_term(x + y, n)
        cauchy_seq(exp_term(x), exp_term(y))(n) = exp_term(x + y)(n)
    }
}

/// The exponential addition law: (x + y).exp = x.exp * y.exp.
theorem exp_add(x: Real, y: Real) {
    (x + y).exp = x.exp * y.exp
} by {
    // Show absolute convergence
    absolutely_converges(exp_term(x))
    absolutely_converges(exp_term(y))

    // Apply Cauchy product theorem
    converges_to(partial(cauchy_seq(exp_term(x), exp_term(y))),
                 limit(partial(exp_term(x))) * limit(partial(exp_term(y))))

    // The Cauchy sequence equals exp_term(x+y)
    cauchy_seq(exp_term(x), exp_term(y)) = exp_term(x + y)

    // Therefore the partial sums are equal
    partial(cauchy_seq(exp_term(x), exp_term(y))) = partial(exp_term(x + y))

    // So partial(exp_term(x+y)) converges to x.exp * y.exp
    converges_to(partial(exp_term(x + y)), limit(partial(exp_term(x))) * limit(partial(exp_term(y))))

    // By uniqueness of limits
    limit(partial(exp_term(x + y))) = limit(partial(exp_term(x))) * limit(partial(exp_term(y)))

    // By definition of Real.exp
    (x + y).exp = limit(partial(exp_term(x + y)))
    x.exp = limit(partial(exp_term(x)))
    y.exp = limit(partial(exp_term(y)))

    (x + y).exp = x.exp * y.exp
}

/// Euler's number e = exp(1).
attributes Real {
    /// Euler's number e = exp(1).
    let e = (Real.1).exp
}

theorem exp_zero_term(k: Nat) {
    k > Nat.0 implies exp_term(Real.0, k) = Real.0
}

theorem exp_zero_partial(k: Nat) {
    partial(exp_term(Real.0), k.suc) = Real.1
} by {
    define p(x: Nat) -> Bool {
        partial(exp_term(Real.0), x.suc) = Real.1
    }

    // Base case
    partial(exp_term(Real.0), Nat.1) = exp_term(Real.0, Nat.0)
    exp_term(Real.0, Nat.0) = Real.0.pow(Nat.0) / Real.from_rat(Rat.from_nat(Nat.0.factorial))
    Real.0.pow(Nat.0) = Real.1
    from_nat[Real](Nat.0.factorial) = Real.from_rat(Rat.from_nat(Nat.0.factorial))
    from_nat[Real](Nat.1) = Real.1
    Real.1 * Real.1 = Real.1
    Real.1 / Real.1 = Real.1
    partial(exp_term(Real.0), Nat.1) = Real.1
    p(Nat.0)

    forall(x: Nat) {
        if p(x) {
            partial(exp_term(Real.0), x.suc) = Real.1
            partial(exp_term(Real.0), x.suc.suc) = partial(exp_term(Real.0), x.suc) + exp_term(Real.0, x.suc)
            x.suc > Nat.0
            exp_term(Real.0, x.suc) = Real.0
            partial(exp_term(Real.0), x.suc.suc) = Real.1
            p(x.suc)
        }
    }

    p(k)
}

theorem exp_zero {
    (Real.0).exp = Real.1
} by {
    // By exp_zero_partial, partial(exp_term(Real.0), k.suc) = Real.1 for all k
    // This means the sequence partial(exp_term(Real.0)) is eventually constant at Real.1
    // Specifically, for all n >= 1: partial(exp_term(Real.0), n) = Real.1

    // Define the tail of the partial sum sequence starting from index 1
    // tail(partial(exp_term(Real.0)), 1) is the constant sequence Real.1
    forall(i: Nat) {
        // partial(exp_term(Real.0), (1 + i)) = partial(exp_term(Real.0), i.suc)
        partial(exp_term(Real.0), i.suc) = Real.1
        tail(partial(exp_term(Real.0)), Nat.1)(i) = partial(exp_term(Real.0))(Nat.1 + i)
        partial(exp_term(Real.0))(Nat.1 + i) = partial(exp_term(Real.0), Nat.1 + i)
        Nat.1 + i = i.suc
        partial(exp_term(Real.0), Nat.1 + i) = Real.1
        tail(partial(exp_term(Real.0)), Nat.1)(i) = Real.1
        constant[Nat, Real](Real.1, i) = Real.1
        tail(partial(exp_term(Real.0)), Nat.1)(i) = constant[Nat, Real](Real.1, i)
    }

    // So the tail is the constant function
    tail(partial(exp_term(Real.0)), Nat.1) = constant[Nat, Real](Real.1)

    // A constant sequence converges to that constant
    converges(constant[Nat, Real](Real.1))
    converges_to(constant[Nat, Real](Real.1), Real.1)

    // So the tail converges to Real.1
    converges_to(tail(partial(exp_term(Real.0)), Nat.1), Real.1)

    // The tail and the full sequence have the same limit
    converges_to(partial(exp_term(Real.0)), Real.1)

    // By definition of limit
    limit(partial(exp_term(Real.0))) = Real.1

    // By definition of Real.exp
    (Real.0).exp = limit(partial(exp_term(Real.0)))
    (Real.0).exp = Real.1
}

/// The zeroth term in the exponential series is always 1.
theorem exp_term_zero_index(x: Real) {
    exp_term(x, Nat.0) = Real.1
} by {
    exp_term(x, Nat.0) = x.pow(Nat.0) / Real.from_rat(Rat.from_nat(Nat.0.factorial))
    x.pow(Nat.0) = Real.1
    Nat.0.factorial = Nat.1
    Rat.from_nat(Nat.1) = Rat.1
    Real.from_rat(Rat.1) = Real.1
    Real.1 / Real.1 = Real.1
}

/// The first term in the exponential series is x.
theorem exp_term_one_index(x: Real) {
    exp_term(x, Nat.1) = x
} by {
    exp_term(x, Nat.1) = x.pow(Nat.1) / Real.from_rat(Rat.from_nat(Nat.1.factorial))
    x.pow(Nat.1) = x
    Nat.1.factorial = Nat.1
    Rat.from_nat(Nat.1) = Rat.1
    Real.from_rat(Rat.1) = Real.1
    x / Real.1 = x
}

/// For positive x, x^n is positive.
theorem pow_pos(x: Real, n: Nat) {
    x > Real.0 implies x.pow(n) > Real.0
} by {
    define p(k: Nat) -> Bool {
        x > Real.0 implies x.pow(k) > Real.0
    }

    // Base case: x^0 = 1 > 0
    if x > Real.0 {
        x.pow(Nat.0) = Real.1
        Real.1 > Real.0
    }
    p(Nat.0)

    // Inductive step: if x^k > 0, then x^(k+1) = x * x^k > 0
    forall(k: Nat) {
        if p(k) {
            if x > Real.0 {
                x.pow(k) > Real.0
                x.pow(k.suc) = x * x.pow(k)
                // Product of positive reals is positive
                x * x.pow(k) > Real.0
                x.pow(k.suc) > Real.0
            }
            p(k.suc)
        }
    }

    p(n)
}

/// For positive x, all exponential terms are positive.
theorem exp_term_pos(x: Real, n: Nat) {
    x > Real.0 implies exp_term(x, n) > Real.0
} by {
    if x > Real.0 {
        // x^n > 0 when x > 0
        x.pow(n) > Real.0
        x.pow(n).is_positive

        // n! > 0 always
        Real.from_rat(Rat.from_nat(n.factorial)) > Real.0
        Real.from_rat(Rat.from_nat(n.factorial)).is_positive
        Real.from_rat(Rat.from_nat(n.factorial)) != Real.0

        // Quotient of positive reals is positive: a/b = a * b^(-1), and b^(-1) > 0 when b > 0
        Real.from_rat(Rat.from_nat(n.factorial)).inverse.is_positive

        // Product of positive reals is positive
        x.pow(n) * Real.from_rat(Rat.from_nat(n.factorial)).inverse > Real.0

        exp_term(x, n) = x.pow(n) / Real.from_rat(Rat.from_nat(n.factorial))
        x.pow(n) / Real.from_rat(Rat.from_nat(n.factorial)) = x.pow(n) * Real.from_rat(Rat.from_nat(n.factorial)).inverse
        exp_term(x, n) > Real.0
    }
}

/// Helper lemma: partial sums with offset are >= base partial sum.
/// If all terms are nonnegative, then partial(f, n + i) >= partial(f, n).
theorem partial_offset_ge(f: Nat -> Real, n: Nat, i: Nat) {
    is_lower_bound(f, Real.0) implies partial(f, n + i) >= partial(f, n)
} by {
    if is_lower_bound(f, Real.0) {
        define p(j: Nat) -> Bool {
            partial(f, n + j) >= partial(f, n)
        }

        // Base case: j = 0
        p(Nat.0)

        // Inductive step
        forall(j: Nat) {
            if p(j) {
                // IH: partial(f, n + j) >= partial(f, n)
                partial(f, n + j) >= partial(f, n)

                // By partial_monotone: partial(f, n + j) <= partial(f, n + j.suc)
                partial(f, n + j) <= partial(f, (n + j).suc)
                (n + j).suc = n + j.suc
                partial(f, n + j) <= partial(f, n + j.suc)

                // Need to show: partial(f, n + j.suc) >= partial(f, n)
                // We have: partial(f, n) <= partial(f, n + j) <= partial(f, n + j.suc)
                // By transitivity:
                partial(f, n) <= partial(f, n + j.suc)
                partial(f, n + j.suc) >= partial(f, n)

                p(j.suc)
            }
        }

        p(i)
        partial(f, n + i) >= partial(f, n)
    }
}

/// For positive x, x.exp > 1.
/// This follows from the fact that x.exp is the limit of partial sums,
/// and each partial sum (for n ≥ 2) is at least 1 + x.
theorem exp_pos_gt_one(x: Real) {
    x > Real.0 implies x.exp > Real.1
} by {
    if x > Real.0 {
        // x.exp = limit(partial(exp_term(x)))
        x.exp = limit(partial(exp_term(x)))

        // The partial sums converge
        converges(partial(exp_term(x)))

        // exp_term(x, 0) = 1 and exp_term(x, 1) = x
        exp_term(x, Nat.0) = Real.1
        exp_term(x, Nat.1) = x

        // partial(exp_term(x), 2) = exp_term(x, 0) + exp_term(x, 1) = 1 + x
        partial(exp_term(x), Nat.0.suc.suc) = partial(exp_term(x), Nat.1) + exp_term(x, Nat.1)
        partial(exp_term(x), Nat.1) = partial(exp_term(x), Nat.0) + exp_term(x, Nat.0)
        partial(exp_term(x), Nat.0) = Real.0
        partial(exp_term(x), Nat.0.suc.suc) = Real.1 + x

        // All exp_term(x, n) are positive, hence nonnegative
        forall(k: Nat) {
            exp_term(x, k) >= Real.0
            Real.0 <= exp_term(x, k)
        }
        is_lower_bound(exp_term(x), Real.0)

        // Use partial_offset_ge to show partial sums stay >= 1 + x for all n >= 2
        forall(i: Nat) {
            partial(exp_term(x), Nat.0.suc.suc + i) >= partial(exp_term(x), Nat.0.suc.suc)
            partial(exp_term(x), Nat.0.suc.suc + i) >= Real.1 + x
            Real.1 + x <= partial(exp_term(x))(Nat.0.suc.suc + i)
        }

        // This gives us eventual_lb(partial(exp_term(x)), 1 + x) with witness n = 2
        // We need to show: forall j >= 2, 1 + x <= partial(exp_term(x))(j)
        // For any j >= 2, write j = 2 + (j - 2), let i = j - 2
        // Then we have: 1 + x <= partial(exp_term(x))(2 + i) by the above
        forall(j: Nat) {
            if Nat.0.suc.suc <= j {
                // j >= 2, so j = 2 + (j - 2)
                // By the forall above with i = j - 2:
                Real.1 + x <= partial(exp_term(x))(j)
            }
        }
        exists(n0: Nat) {
            forall(i: Nat) {
                n0 <= i implies Real.1 + x <= partial(exp_term(x))(i)
            }
        }
        eventual_lb(partial(exp_term(x)), Real.1 + x)

        // By lb_lte_limit: if the sequence converges and is eventually >= 1+x, then limit >= 1+x
        Real.1 + x <= limit(partial(exp_term(x)))
        Real.1 + x <= x.exp

        // Since x > 0, we have 1 + x > 1
        Real.1 + x > Real.1

        // Therefore x.exp > 1
        x.exp > Real.1
    }
}

/// Helper lemma: the second term of the exponential series.
/// For x > 0, exp_term(x, 2) = x^2/2.
theorem exp_term_two(x: Real) {
    exp_term(x, Nat.0.suc.suc) = x.pow(Nat.0.suc.suc) / Real.from_rat(Rat.from_nat(Nat.0.suc.suc.factorial))
}

/// For positive x, the second exponential term is positive.
/// This represents x^2/2! > 0.
theorem exp_term_two_pos(x: Real) {
    x > Real.0 implies exp_term(x, Nat.0.suc.suc) > Real.0
} by {
    if x > Real.0 {
        // Use exp_term_pos with n = 2
        exp_term(x, Nat.0.suc.suc) > Real.0
    }
}

/// For positive x, x.exp > 1 + x.
/// This follows from the fact that the tail of the series starting from x^2/2 is strictly positive.
theorem exp_gt_one_plus_x(x: Real) {
    x > Real.0 implies x.exp > Real.1 + x
} by {
    if x > Real.0 {
        // x.exp = limit(partial(exp_term(x)))
        x.exp = limit(partial(exp_term(x)))

        // The partial sums converge
        converges(partial(exp_term(x)))

        // exp_term(x, 0) = 1, exp_term(x, 1) = x, exp_term(x, 2) > 0
        exp_term(x, Nat.0) = Real.1
        exp_term(x, Nat.1) = x

        // exp_term(x, 2) > 0 when x > 0
        exp_term(x, Nat.0.suc.suc) > Real.0
        let second_term = exp_term(x, Nat.0.suc.suc)
        second_term > Real.0

        // partial(exp_term(x), 3) = 1 + x + exp_term(x, 2)
        partial(exp_term(x), Nat.0.suc.suc.suc) =
            partial(exp_term(x), Nat.0.suc.suc) + exp_term(x, Nat.0.suc.suc)
        partial(exp_term(x), Nat.0.suc.suc) = Real.1 + x
        partial(exp_term(x), Nat.0.suc.suc.suc) = Real.1 + x + second_term

        // Since second_term > 0, we have partial(exp_term(x), 3) > 1 + x
        Real.1 + x + second_term > Real.1 + x
        partial(exp_term(x), Nat.0.suc.suc.suc) > Real.1 + x

        // All exp_term(x, n) are nonnegative
        forall(k: Nat) {
            x > Real.0 implies exp_term(x, k) > Real.0
            exp_term(x, k) >= Real.0
            Real.0 <= exp_term(x, k)
        }
        is_lower_bound(exp_term(x), Real.0)

        // Use partial_offset_ge: for all i, partial(3 + i) >= partial(3)
        forall(i: Nat) {
            partial(exp_term(x), Nat.0.suc.suc.suc + i) >= partial(exp_term(x), Nat.0.suc.suc.suc)
        }

        // So for all n >= 3, partial(exp_term(x), n) >= partial(exp_term(x), 3) = 1 + x + second_term
        let lower_bound = Real.1 + x + second_term
        is_increasing(partial(exp_term(x)))
        forall(j: Nat) {
            if Nat.0.suc.suc.suc <= j {
                // j >= 3, so we can write j = 3 + (j - 3)
                // By the forall above: partial(j) >= partial(3) = lower_bound
                partial(exp_term(x), Nat.0.suc.suc.suc) <= partial(exp_term(x), j)
                lower_bound <= partial(exp_term(x))(j)
            }
        }
        exists(n0: Nat) {
            forall(i: Nat) {
                n0 <= i implies lower_bound <= partial(exp_term(x))(i)
            }
        }
        eventual_lb(partial(exp_term(x)), lower_bound)

        // By lb_lte_limit: if sequence converges and is eventually >= lower_bound, then limit >= lower_bound
        lower_bound <= limit(partial(exp_term(x)))
        x.exp = limit(partial(exp_term(x)))
        limit(partial(exp_term(x))) = x.exp
        lower_bound <= x.exp

        // Since second_term > 0, we have lower_bound = (1 + x) + second_term > 1 + x
        lower_bound > Real.1 + x

        // Therefore x.exp >= lower_bound > 1 + x
        x.exp >= lower_bound
        gt_of_gte_of_gt[Real](x.exp, lower_bound, Real.1 + x)
        x.exp > Real.1 + x
    }
}

/// The exponential function is always positive.
theorem exp_pos(x: Real) {
    x.exp > Real.0
} by {
    if x = Real.0 {
        // (0).exp = 1 > 0
        (Real.0).exp = Real.1
        Real.1 > Real.0
        x.exp > Real.0
    } else {
        if x > Real.0 {
            // For positive x, x.exp > 1 > 0
            exp_pos_gt_one(x)
            x.exp > Real.1
            Real.1 > Real.0
            Real.0 < Real.1
            Real.1 < x.exp
            lt_trans(Real.0, Real.1, x.exp)
            x.exp > Real.0
        } else {
            // For x < 0, we have -x > 0
            x < Real.0
            Real.0 - x > Real.0 - Real.0
            Real.0 - Real.0 = Real.0
            -x > Real.0

            // By exp_pos_gt_one: (-x).exp > 1
            (-x).exp > Real.1

            // By exp_add: x.exp * (-x).exp = (x + (-x)).exp = (0).exp = 1
            (x + (-x)).exp = x.exp * (-x).exp
            x + (-x) = Real.0
            (Real.0).exp = x.exp * (-x).exp
            Real.1 = x.exp * (-x).exp
            x.exp * (-x).exp = Real.1

            // Since (-x).exp > 1 > 0, we have (-x).exp != 0
            exp_pos_gt_one(-x)
            (-x).exp > Real.1
            Real.1 > Real.0
            Real.0 < Real.1
            Real.1 < (-x).exp
            lt_trans(Real.0, Real.1, (-x).exp)
            (-x).exp > Real.0
            (-x).exp != Real.0

            // From x.exp * (-x).exp = 1 and (-x).exp > 0, we get x.exp = 1 / (-x).exp > 0
            x.exp = Real.1 / (-x).exp

            // Since (-x).exp > 0, we have 1 / (-x).exp > 0
            (-x).exp.is_positive
            inverse_pos((-x).exp)
            ((-x).exp).inverse.is_positive
            Real.1 * ((-x).exp).inverse > Real.0
            Real.1 / (-x).exp = Real.1 * ((-x).exp).inverse
            Real.1 / (-x).exp > Real.0

            x.exp > Real.0
        }
    }
}

/// The exponential function is strictly increasing.
/// If x < y, then x.exp < y.exp.
theorem exp_increasing(x: Real, y: Real) {
    x < y implies x.exp < y.exp
} by {
    if x < y {
        // y - x > 0
        y - x > Real.0

        // By exp_pos_gt_one: (y - x).exp > 1
        (y - x).exp > Real.1

        // By exp_add: y.exp = (x + (y - x)).exp = x.exp * (y - x).exp
        x + (y - x) = y
        (x + (y - x)).exp = x.exp * (y - x).exp
        y.exp = x.exp * (y - x).exp

        // Since x.exp > 0 (by exp_pos)
        x.exp > Real.0

        // From x.exp > 0 and 1 < (y - x).exp, multiply both sides by x.exp:
        // x.exp * 1 < x.exp * (y - x).exp
        x.exp > Real.0
        x.exp.is_positive
        Real.1 < (y - x).exp
        lt_mul_pos_left(x.exp, Real.1, (y - x).exp)
        x.exp * Real.1 < x.exp * (y - x).exp
        x.exp * Real.1 = x.exp

        // Therefore: x.exp < x.exp * (y - x).exp = y.exp
        x.exp < x.exp * (y - x).exp
        x.exp * (y - x).exp = y.exp
        x.exp < y.exp
    }
}

/// The exponential function is unbounded.
/// For any real number a, there exists b such that a < b.exp.
theorem exp_unbounded(a: Real) {
    exists(b: Real) {
        a < b.exp
    }
} by {
    if a < Real.1 {
        // For a < 1, use b = 0: (0).exp = 1 > a
        (Real.0).exp = Real.1
        a < (Real.0).exp
    } else {
        // For a >= 1, use b = a: a.exp > 1 + a > a
        Real.1 <= a
        a > Real.0
        a.exp > Real.1 + a
        Real.1 + a > a
        a.exp > a
        a < a.exp
    }
}

/// Three as a real number.
let three = Real.1 + Real.1 + Real.1
/// Strict reciprocal antitone: 0 < a < b implies 1/b < 1/a
theorem recip_antitone_strict(a: Real, b: Real) {
    a < b and a > Real.0 implies b.inverse < a.inverse
} by {
    if a < b and a > Real.0 {
        Real.0 < a
        a > Real.0
        lt_trans(Real.0, a, b)
        Real.0 < b
        b > Real.0
        inverse_on_positive_flips_inequality[Real](a, b)
        b.inverse < a.inverse
    }
}

/// The real value of two is less than the real value of three.
theorem from_nat_2_lt_3 {
    from_nat[Real](Nat.2) < from_nat[Real](Nat.3)
} by {
    Nat.2 < Nat.3
    Int.from_nat(Nat.2) < Int.from_nat(Nat.3)
    Rat.from_int(Int.from_nat(Nat.2)) < Rat.from_int(Int.from_nat(Nat.3))
    Rat.from_nat(Nat.2) < Rat.from_nat(Nat.3)
    from_rat_maintains_lt(Rat.from_nat(Nat.2), Rat.from_nat(Nat.3))
    Real.from_rat(Rat.from_nat(Nat.2)) < Real.from_rat(Rat.from_nat(Nat.3))
    from_nat_is_from_rat(Nat.2)
    from_nat_is_from_rat(Nat.3)
    from_nat[Real](Nat.2) < from_nat[Real](Nat.3)
}

/// The real value of the natural number two is the constant two.
theorem from_nat_two_eq {
    from_nat[Real](Nat.2) = two
} by {
    from_nat_add[Real](Nat.1, Nat.1)
    from_nat[Real](Nat.1 + Nat.1) = from_nat[Real](Nat.1) + from_nat[Real](Nat.1)
    Nat.1 + Nat.1 = Nat.2
    from_nat[Real](Nat.2) = from_nat[Real](Nat.1) + from_nat[Real](Nat.1)
    from_nat[Real](Nat.1) = Real.1
    from_nat[Real](Nat.2) = Real.1 + Real.1
    Real.1 + Real.1 = two
    from_nat[Real](Nat.2) = two
}

/// The reciprocal of two is one half.
theorem one_div_two_eq_half {
    Real.1 / two = Real.one_half
} by {
    two * Real.one_half = Real.1
    Real.1 = two * Real.one_half
    mul_left_cancel(Real.1, two, Real.one_half)
    Real.1 / two = Real.one_half
}

/// One third is less than one half.
theorem one_third_lt_one_half {
    Real.1 / from_nat[Real](Nat.3) < Real.one_half
} by {
    from_nat[Real](Nat.2) < from_nat[Real](Nat.3)
    from_nat[Real](Nat.2) > Real.0
    recip_antitone_strict(from_nat[Real](Nat.2), from_nat[Real](Nat.3))
    from_nat[Real](Nat.3).inverse < from_nat[Real](Nat.2).inverse
    Real.1 / from_nat[Real](Nat.3) = from_nat[Real](Nat.3).inverse
    Real.1 / from_nat[Real](Nat.2) = from_nat[Real](Nat.2).inverse
    Real.1 / from_nat[Real](Nat.3) < Real.1 / from_nat[Real](Nat.2)
    from_nat[Real](Nat.2) = two
    Real.1 / from_nat[Real](Nat.2) = Real.1 / two
    one_div_two_eq_half
    Real.1 / two = Real.one_half
    Real.1 / from_nat[Real](Nat.3) < Real.one_half
}

/// One third plus one half is less than one.
theorem third_plus_half_lt_one {
    Real.1 / from_nat[Real](Nat.3) + Real.one_half < Real.1
} by {
    one_third_lt_one_half
    Real.1 / from_nat[Real](Nat.3) < Real.one_half
    add_lt_lt(Real.1 / from_nat[Real](Nat.3), Real.one_half, Real.one_half, Real.one_half)
    Real.1 / from_nat[Real](Nat.3) + Real.one_half < Real.one_half + Real.one_half
    Real.one_half + Real.one_half = Real.1
    Real.1 / from_nat[Real](Nat.3) + Real.one_half < Real.1
}

/// Two plus one half plus one third is less than three.
theorem bound_lt_three {
    two + Real.one_half + Real.1 / from_nat[Real](Nat.3) < three
} by {
    third_plus_half_lt_one
    Real.1 / from_nat[Real](Nat.3) + Real.one_half < Real.1
    lt_add_right(Real.1 / from_nat[Real](Nat.3) + Real.one_half, Real.1, two)
    (Real.1 / from_nat[Real](Nat.3) + Real.one_half) + two < Real.1 + two
    Real.1 / from_nat[Real](Nat.3) + Real.one_half + two = two + Real.one_half + Real.1 / from_nat[Real](Nat.3)
    two + Real.one_half + Real.1 / from_nat[Real](Nat.3) < Real.1 + two
    Real.1 + two = three
    two + Real.one_half + Real.1 / from_nat[Real](Nat.3) < three
}

/// The real value of six is twice the real value of three.
theorem from_nat_6_eq {
    from_nat[Real](Nat.6) = from_nat[Real](Nat.3) * two
} by {
    Nat.6 = Nat.3 * Nat.2
    from_nat_mul[Real](Nat.3, Nat.2)
    from_nat[Real](Nat.3 * Nat.2) = from_nat[Real](Nat.3) * from_nat[Real](Nat.2)
    from_nat[Real](Nat.6) = from_nat[Real](Nat.3 * Nat.2)
    from_nat_two_eq
    from_nat[Real](Nat.2) = two
    from_nat[Real](Nat.6) = from_nat[Real](Nat.3) * two
}

/// Twice one sixth is one third.
theorem sixth_times_two {
    (Real.1 / from_nat[Real](Nat.6)) * two = Real.1 / from_nat[Real](Nat.3)
} by {
    from_nat_6_eq
    from_nat[Real](Nat.6) = from_nat[Real](Nat.3) * two
    Real.1 / from_nat[Real](Nat.6) = Real.1 / (from_nat[Real](Nat.3) * two)
    suc_pos(Nat.2)
    Real.from_rat(Rat.from_nat(Nat.3)) > Real.0
    from_nat[Real](Nat.3) = Real.from_rat(Rat.from_nat(Nat.3))
    from_nat[Real](Nat.3) > Real.0
    from_nat[Real](Nat.3) != Real.0
    two != Real.0
    div_mul_cancel_right(two, from_nat[Real](Nat.3))
    two / (from_nat[Real](Nat.3) * two) = Real.1 / from_nat[Real](Nat.3)
    two / (from_nat[Real](Nat.3) * two) = (Real.1 / (from_nat[Real](Nat.3) * two)) * two
    (Real.1 / (from_nat[Real](Nat.3) * two)) * two = Real.1 / from_nat[Real](Nat.3)
    (Real.1 / from_nat[Real](Nat.6)) * two = Real.1 / from_nat[Real](Nat.3)
}

/// The factorial of three is six.
theorem factorial_3_eq_6 {
    Nat.3.factorial = Nat.6
} by {
    factorial_zero
    Nat.0.factorial = Nat.1
    factorial_one
    Nat.1.factorial = Nat.1
    factorial_step(Nat.2)
    Nat.3.factorial = Nat.3 * Nat.2.factorial
    Nat.2.factorial = Nat.2 * Nat.1.factorial
    Nat.1.factorial = Nat.1
    Nat.2.factorial = Nat.2 * Nat.1
    Nat.2 * Nat.1 = Nat.2
    Nat.2.factorial = Nat.2
    Nat.3.factorial = Nat.3 * Nat.2
    nat_mul_3_2
    Nat.3 * Nat.2 = Nat.6
    Nat.3.factorial = Nat.6
}

/// The third term of the exponential series at one is one sixth.
theorem exp_one_term_three {
    exp_term(Real.1, Nat.3) = Real.1 / from_nat[Real](Nat.6)
} by {
    exp_term(Real.1, Nat.3) = Real.1.pow(Nat.3) / Real.from_rat(Rat.from_nat(Nat.3.factorial))
    one_pow[Real](Nat.3)
    Real.1.pow(Nat.3) = Real.1
    factorial_3_eq_6
    Nat.3.factorial = Nat.6
    Real.from_rat(Rat.from_nat(Nat.3.factorial)) = from_nat[Real](Nat.3.factorial)
    from_nat[Real](Nat.3.factorial) = from_nat[Real](Nat.6)
    exp_term(Real.1, Nat.3) = Real.1 / from_nat[Real](Nat.6)
}

/// Dividing by two is multiplying by one half.
theorem div_two_is_mul_half(a: Real) {
    a / two = a * Real.one_half
} by {
    one_div_two_eq_half
    Real.1 / two = Real.one_half
    a / two = a * (Real.1 / two)
    a * (Real.1 / two) = a * Real.one_half
    a / two = a * Real.one_half
}

/// The real value of any natural number is nonnegative.
theorem from_nat_real_nonneg(n: Nat) {
    from_nat[Real](n) >= Real.0
} by {
    from_nat_nonneg(n)
    Rat.0 <= Rat.from_nat(n)
    from_rat_maintains_lte(Rat.0, Rat.from_nat(n))
    Real.from_rat(Rat.0) <= Real.from_rat(Rat.from_nat(n))
    Real.from_rat(Rat.0) = Real.0
    from_nat_is_from_rat(n)
    from_nat[Real](n) = Real.from_rat(Rat.from_nat(n))
    Real.0 <= from_nat[Real](n)
}

/// The real value of four plus any natural number is at least two.
theorem from_nat_4j_ge_two(j: Nat) {
    from_nat[Real](Nat.4 + j) >= two
} by {
    from_nat_add[Real](Nat.4, j)
    from_nat[Real](Nat.4 + j) = from_nat[Real](Nat.4) + from_nat[Real](j)
    from_nat_add[Real](Nat.2, Nat.2)
    from_nat[Real](Nat.2 + Nat.2) = from_nat[Real](Nat.2) + from_nat[Real](Nat.2)
    Nat.2 + Nat.2 = Nat.4
    from_nat[Real](Nat.4) = from_nat[Real](Nat.2) + from_nat[Real](Nat.2)
    from_nat_two_eq
    from_nat[Real](Nat.2) = two
    from_nat[Real](Nat.4) = two + two
    from_nat_real_nonneg(j)
    from_nat[Real](j) >= Real.0
    from_nat[Real](Nat.4) = two + two
    two > Real.0
    Real.0 < two
    lte_add_right(two, two, Real.0)
    Real.0 <= two
    two <= two + Real.0
    two + Real.0 = two
    two <= two + two
    two <= from_nat[Real](Nat.4)
    add_lte_add(two, from_nat[Real](Nat.4), Real.0, from_nat[Real](j))
    two + Real.0 <= from_nat[Real](Nat.4) + from_nat[Real](j)
    two + Real.0 = two
    two <= from_nat[Real](Nat.4) + from_nat[Real](j)
    from_nat[Real](Nat.4 + j) >= two
}

/// The second term of the exponential series at one is one half.
theorem exp_one_term_two {
    exp_term(Real.1, Nat.2) = Real.one_half
} by {
    exp_term(Real.1, Nat.2) = Real.1.pow(Nat.2) / Real.from_rat(Rat.from_nat(Nat.2.factorial))
    one_pow[Real](Nat.2)
    Real.1.pow(Nat.2) = Real.1
    factorial_one
    Nat.1.factorial = Nat.1
    factorial_step(Nat.1)
    Nat.2.factorial = Nat.2 * Nat.1.factorial
    Nat.2.factorial = Nat.2 * Nat.1
    Nat.2 * Nat.1 = Nat.2
    Nat.2.factorial = Nat.2
    Real.from_rat(Rat.from_nat(Nat.2.factorial)) = from_nat[Real](Nat.2.factorial)
    from_nat[Real](Nat.2.factorial) = from_nat[Real](Nat.2)
    from_nat_two_eq
    from_nat[Real](Nat.2) = two
    Real.from_rat(Rat.from_nat(Nat.2.factorial)) = two
    Real.1 / Real.from_rat(Rat.from_nat(Nat.2.factorial)) = Real.1 / two
    one_div_two_eq_half
    Real.1 / two = Real.one_half
    exp_term(Real.1, Nat.2) = Real.one_half
}

/// The tail of e's series from index 3 is dominated by a geometric sequence.
theorem exp_one_tail_geom_bound(j: Nat) {
    exp_term(Real.1, Nat.3 + j) <= exp_term(Real.1, Nat.3) * Real.one_half.pow(j)
} by {
    define p(k: Nat) -> Bool {
        exp_term(Real.1, Nat.3 + k) <= exp_term(Real.1, Nat.3) * Real.one_half.pow(k)
    }
    // Base: j = 0
    Nat.3 + Nat.0 = Nat.3
    pow_zero[Real](Real.one_half)
    Real.one_half.pow(Nat.0) = Real.1
    exp_term(Real.1, Nat.3 + Nat.0) = exp_term(Real.1, Nat.3)
    exp_term(Real.1, Nat.3 + Nat.0) <= exp_term(Real.1, Nat.3) * Real.one_half.pow(Nat.0)
    p(Nat.0)
    // Step
    forall(k: Nat) {
        if p(k) {
            exp_term(Real.1, Nat.3 + k) <= exp_term(Real.1, Nat.3) * Real.one_half.pow(k)
            exp_term_mul_recurrence(Real.1, Nat.3 + k)
            exp_term(Real.1, (Nat.3 + k).suc) * Real.from_rat(Rat.from_nat((Nat.3 + k).suc)) = Real.1 * exp_term(Real.1, Nat.3 + k)
            (Nat.3 + k).suc = Nat.4 + k
            exp_term(Real.1, Nat.4 + k) * Real.from_rat(Rat.from_nat(Nat.4 + k)) = Real.1 * exp_term(Real.1, Nat.3 + k)
            Real.1 * exp_term(Real.1, Nat.3 + k) = exp_term(Real.1, Nat.3 + k)
            exp_term(Real.1, Nat.4 + k) * Real.from_rat(Rat.from_nat(Nat.4 + k)) = exp_term(Real.1, Nat.3 + k)
            Real.from_rat(Rat.from_nat(Nat.4 + k)) = from_nat[Real](Nat.4 + k)
            from_nat_4j_ge_two(k)
            from_nat[Real](Nat.4 + k) >= two
            Real.from_rat(Rat.from_nat(Nat.4 + k)) >= two
            exp_term_pos(Real.1, Nat.4 + k)
            exp_term(Real.1, Nat.4 + k) > Real.0
            mul_le_mul_pos_left(two, Real.from_rat(Rat.from_nat(Nat.4 + k)), exp_term(Real.1, Nat.4 + k))
            exp_term(Real.1, Nat.4 + k) * two <= exp_term(Real.1, Nat.4 + k) * Real.from_rat(Rat.from_nat(Nat.4 + k))
            exp_term(Real.1, Nat.4 + k) * Real.from_rat(Rat.from_nat(Nat.4 + k)) = exp_term(Real.1, Nat.3 + k)
            exp_term(Real.1, Nat.4 + k) * two <= exp_term(Real.1, Nat.3 + k)
            div_le_of_mul_le(exp_term(Real.1, Nat.4 + k), two, exp_term(Real.1, Nat.3 + k))
            exp_term(Real.1, Nat.4 + k) <= exp_term(Real.1, Nat.3 + k) / two
            div_two_is_mul_half(exp_term(Real.1, Nat.3 + k))
            exp_term(Real.1, Nat.3 + k) / two = exp_term(Real.1, Nat.3 + k) * Real.one_half
            exp_term(Real.1, Nat.4 + k) <= exp_term(Real.1, Nat.3 + k) * Real.one_half
            mul_le_mul_pos_right(exp_term(Real.1, Nat.3 + k), exp_term(Real.1, Nat.3) * Real.one_half.pow(k), Real.one_half)
            exp_term(Real.1, Nat.3 + k) * Real.one_half <= (exp_term(Real.1, Nat.3) * Real.one_half.pow(k)) * Real.one_half
            (exp_term(Real.1, Nat.3) * Real.one_half.pow(k)) * Real.one_half = exp_term(Real.1, Nat.3) * (Real.one_half.pow(k) * Real.one_half)
            Real.one_half.pow(k) * Real.one_half = Real.one_half * Real.one_half.pow(k)
            pow_suc(Real.one_half, k)
            Real.one_half.pow(k.suc) = Real.one_half * Real.one_half.pow(k)
            Real.one_half.pow(k) * Real.one_half = Real.one_half.pow(k.suc)
            (exp_term(Real.1, Nat.3) * Real.one_half.pow(k)) * Real.one_half = exp_term(Real.1, Nat.3) * Real.one_half.pow(k.suc)
            lte_trans(exp_term(Real.1, Nat.4 + k), exp_term(Real.1, Nat.3 + k) * Real.one_half, exp_term(Real.1, Nat.3) * Real.one_half.pow(k.suc))
            exp_term(Real.1, Nat.4 + k) <= exp_term(Real.1, Nat.3) * Real.one_half.pow(k.suc)
            Nat.3 + k.suc = Nat.4 + k
            exp_term(Real.1, Nat.3 + k.suc) <= exp_term(Real.1, Nat.3) * Real.one_half.pow(k.suc)
            p(k.suc)
        }
        p(k) implies p(k.suc)
    }
    p(Nat.0) and forall(n: Nat) { p(n) implies p(n.suc) }
    alt_induction(p)
    forall(n: Nat) { p(n) }
    p(j)
}

/// One half is less than one.
theorem one_half_lt_one {
    Real.one_half < Real.1
} by {
    one_half_positive
    Real.0 < Real.one_half
    Real.one_half + Real.one_half = Real.1
    lt_add_right(Real.0, Real.one_half, Real.one_half)
    Real.0 + Real.one_half < Real.one_half + Real.one_half
    Real.0 + Real.one_half = Real.one_half
    Real.one_half < Real.1
}

/// The reciprocal of one half is two.
theorem one_div_one_half_eq_two {
    Real.1 / Real.one_half = two
} by {
    Real.one_half != Real.0
    Real.one_half * Real.one_half.inverse = Real.1
    Real.one_half * two = Real.1
    Real.one_half * Real.one_half.inverse = Real.one_half * two
    Real.one_half.inverse = two
    Real.1 / Real.one_half = Real.1 * Real.one_half.inverse
    Real.1 * Real.one_half.inverse = Real.one_half.inverse
    Real.1 / Real.one_half = two
}

/// Partial sums of the powers of one half are bounded by two.
theorem one_half_geom_partial_bound(m: Nat) {
    partial(Real.one_half.pow, m) <= two
} by {
    Real.one_half > Real.0
    one_half_lt_one
    Real.one_half < Real.1
    Real.1 - Real.one_half = Real.one_half
    pos_geom_indirect_upper_bound(Real.one_half, m)
    partial(Real.one_half.pow, m) * (Real.1 - Real.one_half) <= Real.1
    partial(Real.one_half.pow, m) * Real.one_half <= Real.1
    div_le_of_mul_le(partial(Real.one_half.pow, m), Real.one_half, Real.1)
    partial(Real.one_half.pow, m) <= Real.1 / Real.one_half
    one_div_one_half_eq_two
    Real.1 / Real.one_half = two
    partial(Real.one_half.pow, m) <= two
}

/// Partial sums of the tail of e's series from index 3 are bounded by one third.
theorem exp_one_tail_partial_bound(m: Nat) {
    partial(tail(exp_term(Real.1), Nat.3), m) <= Real.1 / from_nat[Real](Nat.3)
} by {
    define g(k: Nat) -> Real {
        exp_term(Real.1, Nat.3) * Real.one_half.pow(k)
    }
    forall(k: Nat) {
        tail(exp_term(Real.1), Nat.3)(k) = exp_term(Real.1, Nat.3 + k)
        exp_one_tail_geom_bound(k)
        exp_term(Real.1, Nat.3 + k) <= exp_term(Real.1, Nat.3) * Real.one_half.pow(k)
        tail(exp_term(Real.1), Nat.3)(k) <= g(k)
    }
    seq_lte(tail(exp_term(Real.1), Nat.3), g)
    partial_sums_seq_lte(tail(exp_term(Real.1), Nat.3), g)
    seq_lte(partial(tail(exp_term(Real.1), Nat.3)), partial(g))
    forall(k: Nat) {
        mul_seq(exp_term(Real.1, Nat.3), Real.one_half.pow)(k) = exp_term(Real.1, Nat.3) * Real.one_half.pow(k)
        g(k) = mul_seq(exp_term(Real.1, Nat.3), Real.one_half.pow)(k)
    }
    partial(g) = partial(mul_seq(exp_term(Real.1, Nat.3), Real.one_half.pow))
    partial_mul_seq_comm(exp_term(Real.1, Nat.3), Real.one_half.pow)
    partial(mul_seq(exp_term(Real.1, Nat.3), Real.one_half.pow)) = mul_seq(exp_term(Real.1, Nat.3), partial(Real.one_half.pow))
    partial(g) = mul_seq(exp_term(Real.1, Nat.3), partial(Real.one_half.pow))
    partial(tail(exp_term(Real.1), Nat.3), m) <= partial(g, m)
    partial(g, m) = mul_seq(exp_term(Real.1, Nat.3), partial(Real.one_half.pow), m)
    partial(g, m) = exp_term(Real.1, Nat.3) * partial(Real.one_half.pow, m)
    one_half_geom_partial_bound(m)
    partial(Real.one_half.pow, m) <= two
    exp_one_term_three
    exp_term(Real.1, Nat.3) = Real.1 / from_nat[Real](Nat.6)
    exp_term_pos(Real.1, Nat.3)
    exp_term(Real.1, Nat.3) > Real.0
    mul_le_mul_pos_left(partial(Real.one_half.pow, m), two, exp_term(Real.1, Nat.3))
    exp_term(Real.1, Nat.3) * partial(Real.one_half.pow, m) <= exp_term(Real.1, Nat.3) * two
    exp_term(Real.1, Nat.3) * two = (Real.1 / from_nat[Real](Nat.6)) * two
    sixth_times_two
    (Real.1 / from_nat[Real](Nat.6)) * two = Real.1 / from_nat[Real](Nat.3)
    exp_term(Real.1, Nat.3) * two = Real.1 / from_nat[Real](Nat.3)
    exp_term(Real.1, Nat.3) * partial(Real.one_half.pow, m) <= Real.1 / from_nat[Real](Nat.3)
    partial(g, m) = exp_term(Real.1, Nat.3) * partial(Real.one_half.pow, m)
    partial(g, m) <= Real.1 / from_nat[Real](Nat.3)
    lte_trans(partial(tail(exp_term(Real.1), Nat.3), m), partial(g, m), Real.1 / from_nat[Real](Nat.3))
    partial(tail(exp_term(Real.1), Nat.3), m) <= Real.1 / from_nat[Real](Nat.3)
}

/// partial(exp_term(Real.1), 3) = 2 + 1/2
theorem partial_three_value {
    partial(exp_term(Real.1), Nat.3) = two + Real.one_half
} by {
    partial(exp_term(Real.1), Nat.1) = exp_term(Real.1, Nat.0)
    exp_term_zero_index(Real.1)
    exp_term(Real.1, Nat.0) = Real.1
    partial(exp_term(Real.1), Nat.1) = Real.1
    partial(exp_term(Real.1), Nat.2) = partial(exp_term(Real.1), Nat.1) + exp_term(Real.1, Nat.1)
    exp_term_one_index(Real.1)
    exp_term(Real.1, Nat.1) = Real.1
    partial(exp_term(Real.1), Nat.2) = Real.1 + Real.1
    Real.1 + Real.1 = two
    partial(exp_term(Real.1), Nat.2) = two
    partial(exp_term(Real.1), Nat.3) = partial(exp_term(Real.1), Nat.2) + exp_term(Real.1, Nat.2)
    exp_one_term_two
    exp_term(Real.1, Nat.2) = Real.one_half
    partial(exp_term(Real.1), Nat.3) = two + Real.one_half
}

/// For all i >= 3, partial sums are bounded by 2 + 1/2 + 1/3
theorem exp_one_partial_bound(i: Nat) {
    Nat.3 <= i implies partial(exp_term(Real.1), i) <= two + Real.one_half + Real.1 / from_nat[Real](Nat.3)
} by {
    if Nat.3 <= i {
        add_sub(i, Nat.3)
        i - Nat.3 + Nat.3 = i
        partial_tail(exp_term(Real.1), Nat.3, i - Nat.3)
        partial(exp_term(Real.1), Nat.3 + (i - Nat.3)) = partial(exp_term(Real.1), Nat.3) + partial(tail(exp_term(Real.1), Nat.3), i - Nat.3)
        partial(exp_term(Real.1), Nat.3 + (i - Nat.3)) = partial(exp_term(Real.1), i)
        partial(exp_term(Real.1), i) = partial(exp_term(Real.1), Nat.3) + partial(tail(exp_term(Real.1), Nat.3), i - Nat.3)
        partial_three_value
        partial(exp_term(Real.1), Nat.3) = two + Real.one_half
        exp_one_tail_partial_bound(i - Nat.3)
        partial(tail(exp_term(Real.1), Nat.3), i - Nat.3) <= Real.1 / from_nat[Real](Nat.3)
        lte_add_left(two + Real.one_half, Real.1 / from_nat[Real](Nat.3), partial(tail(exp_term(Real.1), Nat.3), i - Nat.3))
        two + Real.one_half + partial(tail(exp_term(Real.1), Nat.3), i - Nat.3) <= two + Real.one_half + Real.1 / from_nat[Real](Nat.3)
        lte_trans(partial(exp_term(Real.1), i), two + Real.one_half + partial(tail(exp_term(Real.1), Nat.3), i - Nat.3), two + Real.one_half + Real.1 / from_nat[Real](Nat.3))
        partial(exp_term(Real.1), i) <= two + Real.one_half + Real.1 / from_nat[Real](Nat.3)
    }
}


/// The mathematical constant e is less than 3.
theorem e_less_than_three {
    Real.e < three
} by {
    Real.e = (Real.1).exp
    (Real.1).exp = limit(partial(exp_term(Real.1)))
    exp_term_partial_converges(Real.1)
    converges(partial(exp_term(Real.1)))
    exists(n0: Nat) {
        forall(i: Nat) {
            n0 <= i implies partial(exp_term(Real.1), i) <= two + Real.one_half + Real.1 / from_nat[Real](Nat.3)
        }
    }
    eventual_ub(partial(exp_term(Real.1)), two + Real.one_half + Real.1 / from_nat[Real](Nat.3))
    ub_imp_limit_lte(partial(exp_term(Real.1)), two + Real.one_half + Real.1 / from_nat[Real](Nat.3))
    limit(partial(exp_term(Real.1))) <= two + Real.one_half + Real.1 / from_nat[Real](Nat.3)
    (Real.1).exp <= two + Real.one_half + Real.1 / from_nat[Real](Nat.3)
    bound_lt_three
    two + Real.one_half + Real.1 / from_nat[Real](Nat.3) < three
    lte_lt_trans((Real.1).exp, two + Real.one_half + Real.1 / from_nat[Real](Nat.3), three)
    (Real.1).exp <= two + Real.one_half + Real.1 / from_nat[Real](Nat.3)
    (Real.1).exp < three
    Real.e = (Real.1).exp
    Real.e < three
}

/// e > 2: the partial sum 1 + 1 + 1/2 = 5/2 is below the limit.
theorem e_gt_two {
    Real.e > two
} by {
    Real.e = (Real.1).exp
    (Real.1).exp = limit(partial(exp_term(Real.1)))
    partial_three_value
    partial(exp_term(Real.1), Nat.3) = two + Real.one_half
    // partial sums are increasing (terms are nonnegative)
    forall(k: Nat) {
        exp_term_pos(Real.1, k)
        exp_term(Real.1, k) > Real.0
        exp_term(Real.1, k) >= Real.0
        Real.0 <= exp_term(Real.1, k)
    }
    is_lower_bound(exp_term(Real.1), Real.0)
    nonneg_partial_increasing(exp_term(Real.1))
    is_increasing(partial(exp_term(Real.1)))
    exp_term_partial_converges(Real.1)
    converges(partial(exp_term(Real.1)))
    increasing_convergent_bounded_by_limit(partial(exp_term(Real.1)))
    is_upper_bound(partial(exp_term(Real.1)), limit(partial(exp_term(Real.1))))
    partial(exp_term(Real.1), Nat.3) <= limit(partial(exp_term(Real.1)))
    partial(exp_term(Real.1), Nat.3) <= (Real.1).exp
    two + Real.one_half <= (Real.1).exp
    one_half_positive
    Real.one_half > Real.0
    Real.one_half.is_positive
    lt_add_pos(two, Real.one_half)
    two < two + Real.one_half
    lt_lte_trans(two, two + Real.one_half, (Real.1).exp)
    two < (Real.1).exp
    Real.e > two
}

/// For x >= 0, x.exp >= 1 + x + x^2/2 (partial sum at 3 terms).
theorem exp_ge_one_plus_x_plus_x2_half(x: Real) {
    x >= Real.0 implies x.exp >= Real.1 + x + x.pow(Nat.2) / two
} by {
    if x >= Real.0 {
        x.exp = limit(partial(exp_term(x)))
        exp_term_partial_converges(x)
        converges(partial(exp_term(x)))
        // partial sums are increasing
        forall(k: Nat) {
            pow_nonneg(x, k)
            Real.0 <= x.pow(k)
            factorial_pos(k)
            Real.from_rat(Rat.from_nat(k.factorial)) > Real.0
            exp_term(x, k) = x.pow(k) / Real.from_rat(Rat.from_nat(k.factorial))
            Real.from_rat(Rat.from_nat(k.factorial)) > Real.0
            Real.from_rat(Rat.from_nat(k.factorial)) != Real.0
            x.pow(k) / Real.from_rat(Rat.from_nat(k.factorial)) = x.pow(k) * Real.from_rat(Rat.from_nat(k.factorial)).inverse
            inverse_pos(Real.from_rat(Rat.from_nat(k.factorial)))
            Real.from_rat(Rat.from_nat(k.factorial)).inverse.is_positive
            Real.from_rat(Rat.from_nat(k.factorial)).inverse >= Real.0
            x.pow(k) >= Real.0
            Real.from_rat(Rat.from_nat(k.factorial)).inverse >= Real.0
            mul_nonneg(x.pow(k), Real.from_rat(Rat.from_nat(k.factorial)).inverse)
            x.pow(k) * Real.from_rat(Rat.from_nat(k.factorial)).inverse >= Real.0
            Real.0 <= x.pow(k) * Real.from_rat(Rat.from_nat(k.factorial)).inverse
            x.pow(k) / Real.from_rat(Rat.from_nat(k.factorial)) >= Real.0
            exp_term(x, k) >= Real.0
            Real.0 <= exp_term(x, k)
        }
        is_lower_bound(exp_term(x), Real.0)
        nonneg_partial_increasing(exp_term(x))
        is_increasing(partial(exp_term(x)))
        increasing_convergent_bounded_by_limit(partial(exp_term(x)))
        is_upper_bound(partial(exp_term(x)), limit(partial(exp_term(x))))
        partial(exp_term(x), Nat.3) <= limit(partial(exp_term(x)))
        partial(exp_term(x), Nat.3) <= x.exp
        // partial(exp_term(x), 3) = 1 + x + x^2/2
        partial(exp_term(x), Nat.1) = exp_term(x, Nat.0)
        exp_term_zero_index(x)
        exp_term(x, Nat.0) = Real.1
        partial(exp_term(x), Nat.1) = Real.1
        partial(exp_term(x), Nat.2) = partial(exp_term(x), Nat.1) + exp_term(x, Nat.1)
        exp_term_one_index(x)
        exp_term(x, Nat.1) = x
        partial(exp_term(x), Nat.2) = Real.1 + x
        partial(exp_term(x), Nat.3) = partial(exp_term(x), Nat.2) + exp_term(x, Nat.2)
        exp_term(x, Nat.2) = x.pow(Nat.2) / Real.from_rat(Rat.from_nat(Nat.2.factorial))
        factorial_one
        Nat.1.factorial = Nat.1
        factorial_step(Nat.1)
        Nat.2.factorial = Nat.2 * Nat.1.factorial
        Nat.2.factorial = Nat.2 * Nat.1
        Nat.2 * Nat.1 = Nat.2
        Nat.2.factorial = Nat.2
        Real.from_rat(Rat.from_nat(Nat.2.factorial)) = from_nat[Real](Nat.2.factorial)
        from_nat[Real](Nat.2.factorial) = from_nat[Real](Nat.2)
        from_nat[Real](Nat.2) = two
        Real.from_rat(Rat.from_nat(Nat.2.factorial)) = two
        exp_term(x, Nat.2) = x.pow(Nat.2) / two
        partial(exp_term(x), Nat.3) = Real.1 + x + x.pow(Nat.2) / two
        Real.1 + x + x.pow(Nat.2) / two <= x.exp
        x.exp >= Real.1 + x + x.pow(Nat.2) / two
    }
}

// TODO: Prove that e is not an integer.
// // The mathematical constant e is not an integer.
// // Proof sketch: we already know 2 < e < 3 (e_gt_two and e_less_than_three),
// // and no integer lies strictly between 2 and 3. Formally this needs a
// // predicate "x is an integer real" (e.g. exists n: Int with Real.from_int(n) = x)
// // plus the fact that Real.from_int is injective and order-preserving, and
// // that there is no integer strictly between 2 and 3.
// theorem e_not_integer {
//     not (exists(n: Int) { Real.from_int(n) = Real.e })
// }
