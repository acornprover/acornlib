from nat import Nat, mul_suc_right, lte_mul_both, pow_add, add_sub, lte_add_right, add_cancels_right, add_cancels_left, distrib_left, mul_comm, from_nat, add_one_right, suc_sub_one
from rat import Rat
from list import partial, partial_scalar_mul, partial_pointwise_eq, partial_split_last, partial_one, partial_add
from algebra.semigroup import mul_fn
from algebra.add_semigroup import add_fn
from algebra.add_comm_group import sub_add_sub
from real.real_field import Real, mul_div
from real.real_ring import converges, limit, converges_to, mul_abs, mul_zero_left, mul_zero_right, mul_neg_one_left, mul_neg_right, real_mul_comm
from real.real_seq import eq_imp_limit, eventual_eq, converges_to_imp_converges, converges_imp_converges_to, converges_to_unique, add_seq, limit_add_seq
from real.real_base import abs_neg, neg_neg, add_comm, add_assoc
from real.abs_conv import absolutely_converges, abs_fn, absolutely_converges_imp_converges, sub_seq
from real.real_series import partial_suc, partial_zero, tail, tail_imp_converges_to, const_converges, mul_seq, neg_seq, limit_neg_seq, partial_add_seq_comm
from real.exp import exp_term, factorial_pos, pow_suc, choose_factorial_inverse, binomial_fraction_transform, div_fn, partial_div_fn, function_extensionality_on_range, exp_term_partial_converges, mul_assoc_real, mul_frac_assoc, exp_term_product_binomial
from real.trig import sin_term, cos_term, sin_term_abs_converges, cos_term_abs_converges, two_mul_suc, alternating_sign_abs, cos_zero, cos_neg, sin_neg
from real.cauchy import cauchy_product, cauchy_seq, cauchy_coefficient, cauchy_product_converges, cauchy_product_zero
from real.double_sum import limit_sub_seq
from algebra.ring.ring import alternating_sign, alternating_sign_zero, alternating_sign_suc, alternating_sign_eq_neg_one_pow
from comm_ring import binomial_term, binomial

numerals Real
numerals Nat

/// The alternating sign of a sum is the product of the alternating signs.
theorem alternating_sign_mul(a: Nat, b: Nat) {
    alternating_sign[Real](a) * alternating_sign[Real](b) = alternating_sign[Real](a + b)
} by {
    alternating_sign_eq_neg_one_pow[Real](a)
    alternating_sign[Real](a) = (-Real.1).pow(a)
    alternating_sign_eq_neg_one_pow[Real](b)
    alternating_sign[Real](b) = (-Real.1).pow(b)
    alternating_sign[Real](a) * alternating_sign[Real](b) = (-Real.1).pow(a) * (-Real.1).pow(b)
    pow_add[Real](-Real.1, a, b)
    (-Real.1).pow(a) * (-Real.1).pow(b) = (-Real.1).pow(a + b)
    alternating_sign_eq_neg_one_pow[Real](a + b)
    alternating_sign[Real](a + b) = (-Real.1).pow(a + b)
    alternating_sign[Real](a) * alternating_sign[Real](b) = alternating_sign[Real](a + b)
}

/// The double of a difference is the difference of doubles.
theorem two_mul_sub(n: Nat, k: Nat) {
    k <= n implies Nat.2 * (n - k) = Nat.2 * n - Nat.2 * k
} by {
    if k <= n {
        add_sub(n, k)
        n - k + k = n
        Nat.2 * (n - k + k) = Nat.2 * n
        Nat.2 * (n - k + k) = Nat.2 * (n - k) + Nat.2 * k
        Nat.2 * (n - k) + Nat.2 * k = Nat.2 * n
        lte_mul_both(Nat.2, k, n)
        Nat.2 * k <= Nat.2 * n
        add_sub(Nat.2 * n, Nat.2 * k)
        Nat.2 * n - Nat.2 * k + Nat.2 * k = Nat.2 * n
        Nat.2 * (n - k) + Nat.2 * k = Nat.2 * n - Nat.2 * k + Nat.2 * k
        add_cancels_right(Nat.2 * k, Nat.2 * (n - k), Nat.2 * n - Nat.2 * k)
        Nat.2 * (n - k) = Nat.2 * n - Nat.2 * k
    }
}

/// The subsequence of f at the even indices.
define even_subseq(f: Nat -> Real, k: Nat) -> Real {
    f(Nat.2 * k)
}

/// The subsequence of f at the odd indices.
define odd_subseq(f: Nat -> Real, k: Nat) -> Real {
    f(Nat.2 * k + Nat.1)
}

/// A partial sum of 2(n+1) terms splits into its even- and odd-indexed parts.
theorem partial_split_even(f: Nat -> Real, n: Nat) {
    partial(f, Nat.2 * n.suc) = partial(even_subseq(f), n.suc) + partial(odd_subseq(f), n.suc)
} by {
    define p(k: Nat) -> Bool {
        partial(f, Nat.2 * k.suc) = partial(even_subseq(f), k.suc) + partial(odd_subseq(f), k.suc)
    }

    // Base case: two terms split into even and odd.
    partial_split_last(f, Nat.1)
    partial(f, Nat.2) = partial(f, Nat.1) + f(Nat.1)
    partial_one(f)
    partial(f, Nat.1) = f(Nat.0)
    partial(f, Nat.2) = f(Nat.0) + f(Nat.1)
    partial_split_last(even_subseq(f), Nat.0)
    partial(even_subseq(f), Nat.1) = partial(even_subseq(f), Nat.0) + even_subseq(f)(Nat.0)
    partial_zero(even_subseq(f))
    partial(even_subseq(f), Nat.0) = Real.0
    partial(even_subseq(f), Nat.1) = even_subseq(f)(Nat.0)
    partial_split_last(odd_subseq(f), Nat.0)
    partial(odd_subseq(f), Nat.1) = partial(odd_subseq(f), Nat.0) + odd_subseq(f)(Nat.0)
    partial_zero(odd_subseq(f))
    partial(odd_subseq(f), Nat.1) = odd_subseq(f)(Nat.0)
    p(Nat.0)

    // Inductive step.
    forall(k: Nat) {
        if p(k) {
            two_mul_suc(k.suc)
            Nat.2 * k.suc.suc = (Nat.2 * k.suc).suc.suc
            partial(f, Nat.2 * k.suc.suc) = partial(f, (Nat.2 * k.suc).suc.suc)
            partial_split_last(f, (Nat.2 * k.suc).suc)
            partial(f, (Nat.2 * k.suc).suc.suc) = partial(f, (Nat.2 * k.suc).suc) + f((Nat.2 * k.suc).suc)
            partial_split_last(f, Nat.2 * k.suc)
            partial(f, (Nat.2 * k.suc).suc) = partial(f, Nat.2 * k.suc) + f(Nat.2 * k.suc)
            partial(f, Nat.2 * k.suc.suc) = partial(f, Nat.2 * k.suc) + f(Nat.2 * k.suc) + f((Nat.2 * k.suc).suc)
            p(k)
            partial(f, Nat.2 * k.suc) = partial(even_subseq(f), k.suc) + partial(odd_subseq(f), k.suc)
            partial(f, Nat.2 * k.suc.suc) = partial(even_subseq(f), k.suc) + partial(odd_subseq(f), k.suc) + f(Nat.2 * k.suc) + f((Nat.2 * k.suc).suc)
            partial_split_last(even_subseq(f), k.suc)
            partial(even_subseq(f), k.suc.suc) = partial(even_subseq(f), k.suc) + even_subseq(f)(k.suc)
            even_subseq(f)(k.suc) = f(Nat.2 * k.suc)
            partial(even_subseq(f), k.suc.suc) = partial(even_subseq(f), k.suc) + f(Nat.2 * k.suc)
            partial_split_last(odd_subseq(f), k.suc)
            partial(odd_subseq(f), k.suc.suc) = partial(odd_subseq(f), k.suc) + odd_subseq(f)(k.suc)
            odd_subseq(f)(k.suc) = f(Nat.2 * k.suc + Nat.1)
            (Nat.2 * k.suc).suc = Nat.2 * k.suc + Nat.1
            partial(odd_subseq(f), k.suc.suc) = partial(odd_subseq(f), k.suc) + f((Nat.2 * k.suc).suc)
            partial(even_subseq(f), k.suc.suc) + partial(odd_subseq(f), k.suc.suc) =
                (partial(even_subseq(f), k.suc) + f(Nat.2 * k.suc)) + (partial(odd_subseq(f), k.suc) + f((Nat.2 * k.suc).suc))
            partial(even_subseq(f), k.suc.suc) + partial(odd_subseq(f), k.suc.suc) =
                partial(even_subseq(f), k.suc) + f(Nat.2 * k.suc) + partial(odd_subseq(f), k.suc) + f((Nat.2 * k.suc).suc)
            add_assoc(partial(even_subseq(f), k.suc), partial(odd_subseq(f), k.suc), f(Nat.2 * k.suc))
            partial(even_subseq(f), k.suc) + partial(odd_subseq(f), k.suc) + f(Nat.2 * k.suc) =
                partial(even_subseq(f), k.suc) + (partial(odd_subseq(f), k.suc) + f(Nat.2 * k.suc))
            add_comm(partial(odd_subseq(f), k.suc), f(Nat.2 * k.suc))
            partial(odd_subseq(f), k.suc) + f(Nat.2 * k.suc) = f(Nat.2 * k.suc) + partial(odd_subseq(f), k.suc)
            partial(even_subseq(f), k.suc) + partial(odd_subseq(f), k.suc) + f(Nat.2 * k.suc) =
                partial(even_subseq(f), k.suc) + (f(Nat.2 * k.suc) + partial(odd_subseq(f), k.suc))
            add_assoc(partial(even_subseq(f), k.suc), f(Nat.2 * k.suc), partial(odd_subseq(f), k.suc))
            partial(even_subseq(f), k.suc) + f(Nat.2 * k.suc) + partial(odd_subseq(f), k.suc) =
                partial(even_subseq(f), k.suc) + (f(Nat.2 * k.suc) + partial(odd_subseq(f), k.suc))
            partial(even_subseq(f), k.suc) + partial(odd_subseq(f), k.suc) + f(Nat.2 * k.suc) =
                partial(even_subseq(f), k.suc) + f(Nat.2 * k.suc) + partial(odd_subseq(f), k.suc)
            partial(even_subseq(f), k.suc) + partial(odd_subseq(f), k.suc) + f(Nat.2 * k.suc) + f((Nat.2 * k.suc).suc) =
                partial(even_subseq(f), k.suc) + f(Nat.2 * k.suc) + partial(odd_subseq(f), k.suc) + f((Nat.2 * k.suc).suc)
            partial(f, Nat.2 * k.suc.suc) = partial(even_subseq(f), k.suc.suc) + partial(odd_subseq(f), k.suc.suc)
            p(k.suc)
        }
    }

    p(Nat.0) and forall(k: Nat) { p(k) implies p(k.suc) }
    Nat.induction(p)
    p(n)
}

/// Swapping the second and fourth summands of a four-term sum.
theorem add_swap_outers(a: Real, b: Real, c: Real, d: Real) {
    a + b + c + d = a + d + b + c
} by {
    add_assoc(a + b, c, d)
    (a + b) + c + d = (a + b) + (c + d)
    add_comm(c, d)
    c + d = d + c
    (a + b) + (c + d) = (a + b) + (d + c)
    add_assoc(a + b, d, c)
    (a + b) + d + c = (a + b) + (d + c)
    (a + b) + (c + d) = (a + b) + d + c
    add_assoc(a, b, d)
    a + b + d = a + (b + d)
    add_comm(b, d)
    b + d = d + b
    a + b + d = a + (d + b)
    add_assoc(a, d, b)
    a + d + b = a + (d + b)
    a + b + d = a + d + b
    a + b + d + c = a + d + b + c
    (a + b) + (c + d) = a + d + b + c
    a + b + c + d = a + d + b + c
}

/// A partial sum of 2n+1 terms splits into its even- and odd-indexed parts.
theorem partial_split_odd(f: Nat -> Real, n: Nat) {
    partial(f, Nat.2 * n + Nat.1) = partial(even_subseq(f), n.suc) + partial(odd_subseq(f), n)
} by {
    define p(k: Nat) -> Bool {
        partial(f, Nat.2 * k + Nat.1) = partial(even_subseq(f), k.suc) + partial(odd_subseq(f), k)
    }

    // Base case: one term is the zeroth even term.
    partial_one(f)
    partial(f, Nat.1) = f(Nat.0)
    partial_split_last(even_subseq(f), Nat.0)
    partial(even_subseq(f), Nat.1) = partial(even_subseq(f), Nat.0) + even_subseq(f)(Nat.0)
    partial_zero(even_subseq(f))
    partial(even_subseq(f), Nat.0) = Real.0
    even_subseq(f)(Nat.0) = f(Nat.0)
    partial(even_subseq(f), Nat.1) = f(Nat.0)
    partial_zero(odd_subseq(f))
    partial(odd_subseq(f), Nat.0) = Real.0
    p(Nat.0)

    // Inductive step.
    forall(k: Nat) {
        if p(k) {
            Nat.2 * k.suc + Nat.1 = (Nat.2 * k + Nat.1).suc.suc
            partial(f, Nat.2 * k.suc + Nat.1) = partial(f, (Nat.2 * k + Nat.1).suc.suc)
            partial_split_last(f, (Nat.2 * k + Nat.1).suc)
            partial(f, (Nat.2 * k + Nat.1).suc.suc) = partial(f, (Nat.2 * k + Nat.1).suc) + f((Nat.2 * k + Nat.1).suc)
            partial_split_last(f, Nat.2 * k + Nat.1)
            partial(f, (Nat.2 * k + Nat.1).suc) = partial(f, Nat.2 * k + Nat.1) + f(Nat.2 * k + Nat.1)
            partial(f, Nat.2 * k.suc + Nat.1) = partial(f, Nat.2 * k + Nat.1) + f(Nat.2 * k + Nat.1) + f((Nat.2 * k + Nat.1).suc)
            p(k)
            partial(f, Nat.2 * k + Nat.1) = partial(even_subseq(f), k.suc) + partial(odd_subseq(f), k)
            partial(f, Nat.2 * k.suc + Nat.1) = partial(even_subseq(f), k.suc) + partial(odd_subseq(f), k) + f(Nat.2 * k + Nat.1) + f((Nat.2 * k + Nat.1).suc)
            partial_split_last(even_subseq(f), k.suc)
            partial(even_subseq(f), k.suc.suc) = partial(even_subseq(f), k.suc) + even_subseq(f)(k.suc)
            even_subseq(f)(k.suc) = f(Nat.2 * k.suc)
            two_mul_suc(k)
            Nat.2 * k.suc = (Nat.2 * k).suc.suc
            even_subseq(f)(k.suc) = f((Nat.2 * k).suc.suc)
            partial(even_subseq(f), k.suc.suc) = partial(even_subseq(f), k.suc) + f((Nat.2 * k).suc.suc)
            partial_split_last(odd_subseq(f), k)
            partial(odd_subseq(f), k.suc) = partial(odd_subseq(f), k) + odd_subseq(f)(k)
            odd_subseq(f)(k) = f(Nat.2 * k + Nat.1)
            partial(odd_subseq(f), k.suc) = partial(odd_subseq(f), k) + f(Nat.2 * k + Nat.1)
            partial(even_subseq(f), k.suc.suc) + partial(odd_subseq(f), k.suc) =
                (partial(even_subseq(f), k.suc) + f((Nat.2 * k).suc.suc)) + (partial(odd_subseq(f), k) + f(Nat.2 * k + Nat.1))
            partial(even_subseq(f), k.suc.suc) + partial(odd_subseq(f), k.suc) =
                partial(even_subseq(f), k.suc) + f((Nat.2 * k).suc.suc) + partial(odd_subseq(f), k) + f(Nat.2 * k + Nat.1)
            (Nat.2 * k).suc.suc = (Nat.2 * k + Nat.1).suc
            partial(f, Nat.2 * k.suc + Nat.1) =
                partial(even_subseq(f), k.suc) + partial(odd_subseq(f), k) + f(Nat.2 * k + Nat.1) + f((Nat.2 * k).suc.suc)
            add_swap_outers(partial(even_subseq(f), k.suc), partial(odd_subseq(f), k), f(Nat.2 * k + Nat.1), f((Nat.2 * k).suc.suc))
            partial(even_subseq(f), k.suc) + partial(odd_subseq(f), k) + f(Nat.2 * k + Nat.1) + f((Nat.2 * k).suc.suc) =
                partial(even_subseq(f), k.suc) + f((Nat.2 * k).suc.suc) + partial(odd_subseq(f), k) + f(Nat.2 * k + Nat.1)
            partial(f, Nat.2 * k.suc + Nat.1) =
                partial(even_subseq(f), k.suc) + f((Nat.2 * k).suc.suc) + partial(odd_subseq(f), k) + f(Nat.2 * k + Nat.1)
            partial(f, Nat.2 * k.suc + Nat.1) = partial(even_subseq(f), k.suc.suc) + partial(odd_subseq(f), k.suc)
            p(k.suc)
        }
    }

    p(Nat.0) and forall(k: Nat) { p(k) implies p(k.suc) }
    Nat.induction(p)
    p(n)
}


/// The product of two signed monomials combines the signs.
theorem signed_mul_combine(s: Real, t: Real, a: Real, b: Real) {
    (s * a) * (t * b) = (s * t) * (a * b)
} by {
    mul_assoc_real(s * a, t, b)
    ((s * a) * t) * b = (s * a) * (t * b)
    (s * a) * (t * b) = ((s * a) * t) * b
    mul_assoc_real(s, a, t)
    (s * a) * t = s * (a * t)
    real_mul_comm(a, t)
    a * t = t * a
    (s * a) * t = s * (t * a)
    mul_assoc_real(s, t, a)
    (s * t) * a = s * (t * a)
    (s * a) * t = (s * t) * a
    ((s * a) * t) * b = ((s * t) * a) * b
    mul_assoc_real(s * t, a, b)
    ((s * t) * a) * b = (s * t) * (a * b)
    (s * a) * (t * b) = (s * t) * (a * b)
}

/// Summing two fractions with the same denominator and a shared factor.
theorem div_add_same_denom(a: Real, x: Real, y: Real, f: Real) {
    f != Real.0 implies a * x / f + a * y / f = a * (x + y) / f
} by {
    if f != Real.0 {
        a * x / f = a * x * f.inverse
        a * y / f = a * y * f.inverse
        a * x * f.inverse + a * y * f.inverse = a * (x + y) * f.inverse
        a * (x + y) / f = a * (x + y) * f.inverse
        a * x / f + a * y / f = a * (x + y) / f
    }
}

/// The product of two alternating-signed power-over-factorial terms is a
/// binomial term of the combined row.
theorem signed_terms_binom_product(x: Real, y: Real, a: Nat, b: Nat, e: Nat, f: Nat) {
    (alternating_sign[Real](a) * x.pow(Nat.2 * a + e) / Real.from_rat(Rat.from_nat((Nat.2 * a + e).factorial))) *
    (alternating_sign[Real](b) * y.pow(Nat.2 * b + f) / Real.from_rat(Rat.from_nat((Nat.2 * b + f).factorial))) =
        alternating_sign[Real](a + b) * binomial_term[Real](x, y, Nat.2 * a + e + Nat.2 * b + f, Nat.2 * a + e) /
        Real.from_rat(Rat.from_nat((Nat.2 * a + e + Nat.2 * b + f).factorial))
} by {
    // Unfold the two terms and multiply the fractions.
    let f1 = Real.from_rat(Rat.from_nat((Nat.2 * a + e).factorial))
    let f2 = Real.from_rat(Rat.from_nat((Nat.2 * b + f).factorial))
    f1 != Real.0
    f2 != Real.0
    mul_div(alternating_sign[Real](a) * x.pow(Nat.2 * a + e), f1, alternating_sign[Real](b) * y.pow(Nat.2 * b + f), f2)
    (alternating_sign[Real](a) * x.pow(Nat.2 * a + e)) / f1 * ((alternating_sign[Real](b) * y.pow(Nat.2 * b + f)) / f2) =
        (alternating_sign[Real](a) * x.pow(Nat.2 * a + e)) * (alternating_sign[Real](b) * y.pow(Nat.2 * b + f)) / (f1 * f2)
    (alternating_sign[Real](a) * x.pow(Nat.2 * a + e)) * (alternating_sign[Real](b) * y.pow(Nat.2 * b + f)) =
        alternating_sign[Real](a) * alternating_sign[Real](b) * x.pow(Nat.2 * a + e) * y.pow(Nat.2 * b + f)
    (alternating_sign[Real](a) * x.pow(Nat.2 * a + e)) * (alternating_sign[Real](b) * y.pow(Nat.2 * b + f)) / (f1 * f2) =
        alternating_sign[Real](a) * alternating_sign[Real](b) * x.pow(Nat.2 * a + e) * y.pow(Nat.2 * b + f) / (f1 * f2)
    alternating_sign_mul(a, b)
    alternating_sign[Real](a) * alternating_sign[Real](b) = alternating_sign[Real](a + b)
    alternating_sign[Real](a) * alternating_sign[Real](b) * x.pow(Nat.2 * a + e) * y.pow(Nat.2 * b + f) =
        alternating_sign[Real](a + b) * x.pow(Nat.2 * a + e) * y.pow(Nat.2 * b + f)
    (alternating_sign[Real](a) * x.pow(Nat.2 * a + e)) * (alternating_sign[Real](b) * y.pow(Nat.2 * b + f)) / (f1 * f2) =
        alternating_sign[Real](a + b) * x.pow(Nat.2 * a + e) * y.pow(Nat.2 * b + f) / (f1 * f2)

    // Convert to a binomial term of the combined row.
    Nat.2 * a + e <= Nat.2 * a + e + Nat.2 * b + f
    binomial_fraction_transform(x, y, Nat.2 * a + e + Nat.2 * b + f, Nat.2 * a + e)
    (x.pow(Nat.2 * a + e) * y.pow(Nat.2 * a + e + Nat.2 * b + f - (Nat.2 * a + e))) /
        (Real.from_rat(Rat.from_nat((Nat.2 * a + e).factorial)) * Real.from_rat(Rat.from_nat((Nat.2 * a + e + Nat.2 * b + f - (Nat.2 * a + e)).factorial))) =
        (from_nat[Real]((Nat.2 * a + e + Nat.2 * b + f).binom(Nat.2 * a + e)) * x.pow(Nat.2 * a + e) * y.pow(Nat.2 * a + e + Nat.2 * b + f - (Nat.2 * a + e))) /
        Real.from_rat(Rat.from_nat((Nat.2 * a + e + Nat.2 * b + f).factorial))
    Nat.2 * a + e + Nat.2 * b + f - (Nat.2 * a + e) = Nat.2 * b + f
    (x.pow(Nat.2 * a + e) * y.pow(Nat.2 * b + f)) /
        (Real.from_rat(Rat.from_nat((Nat.2 * a + e).factorial)) * Real.from_rat(Rat.from_nat((Nat.2 * b + f).factorial))) =
        (x.pow(Nat.2 * a + e) * y.pow(Nat.2 * a + e + Nat.2 * b + f - (Nat.2 * a + e))) /
        (Real.from_rat(Rat.from_nat((Nat.2 * a + e).factorial)) * Real.from_rat(Rat.from_nat((Nat.2 * a + e + Nat.2 * b + f - (Nat.2 * a + e)).factorial)))
    (x.pow(Nat.2 * a + e) * y.pow(Nat.2 * b + f)) /
        (Real.from_rat(Rat.from_nat((Nat.2 * a + e).factorial)) * Real.from_rat(Rat.from_nat((Nat.2 * b + f).factorial))) =
        (from_nat[Real]((Nat.2 * a + e + Nat.2 * b + f).binom(Nat.2 * a + e)) * x.pow(Nat.2 * a + e) * y.pow(Nat.2 * b + f)) /
        Real.from_rat(Rat.from_nat((Nat.2 * a + e + Nat.2 * b + f).factorial))
    alternating_sign[Real](a + b) * ((x.pow(Nat.2 * a + e) * y.pow(Nat.2 * b + f)) / (f1 * f2)) =
        alternating_sign[Real](a + b) * ((from_nat[Real]((Nat.2 * a + e + Nat.2 * b + f).binom(Nat.2 * a + e)) * x.pow(Nat.2 * a + e) * y.pow(Nat.2 * b + f)) /
        Real.from_rat(Rat.from_nat((Nat.2 * a + e + Nat.2 * b + f).factorial)))
    f1 * f2 != Real.0
    Real.from_rat(Rat.from_nat((Nat.2 * a + e + Nat.2 * b + f).factorial)) != Real.0
    mul_frac_assoc(alternating_sign[Real](a + b), x.pow(Nat.2 * a + e) * y.pow(Nat.2 * b + f), f1 * f2)
    alternating_sign[Real](a + b) * ((x.pow(Nat.2 * a + e) * y.pow(Nat.2 * b + f)) / (f1 * f2)) =
        (alternating_sign[Real](a + b) * (x.pow(Nat.2 * a + e) * y.pow(Nat.2 * b + f))) / (f1 * f2)
    mul_frac_assoc(alternating_sign[Real](a + b), from_nat[Real]((Nat.2 * a + e + Nat.2 * b + f).binom(Nat.2 * a + e)) * x.pow(Nat.2 * a + e) * y.pow(Nat.2 * b + f), Real.from_rat(Rat.from_nat((Nat.2 * a + e + Nat.2 * b + f).factorial)))
    alternating_sign[Real](a + b) * ((from_nat[Real]((Nat.2 * a + e + Nat.2 * b + f).binom(Nat.2 * a + e)) * x.pow(Nat.2 * a + e) * y.pow(Nat.2 * b + f)) /
        Real.from_rat(Rat.from_nat((Nat.2 * a + e + Nat.2 * b + f).factorial))) =
        (alternating_sign[Real](a + b) * (from_nat[Real]((Nat.2 * a + e + Nat.2 * b + f).binom(Nat.2 * a + e)) * x.pow(Nat.2 * a + e) * y.pow(Nat.2 * b + f))) /
        Real.from_rat(Rat.from_nat((Nat.2 * a + e + Nat.2 * b + f).factorial))
    alternating_sign[Real](a + b) * (x.pow(Nat.2 * a + e) * y.pow(Nat.2 * b + f)) / (f1 * f2) =
        alternating_sign[Real](a + b) * (from_nat[Real]((Nat.2 * a + e + Nat.2 * b + f).binom(Nat.2 * a + e)) * x.pow(Nat.2 * a + e) * y.pow(Nat.2 * b + f)) /
        Real.from_rat(Rat.from_nat((Nat.2 * a + e + Nat.2 * b + f).factorial))
    mul_assoc_real(alternating_sign[Real](a + b), x.pow(Nat.2 * a + e), y.pow(Nat.2 * b + f))
    alternating_sign[Real](a + b) * x.pow(Nat.2 * a + e) * y.pow(Nat.2 * b + f) =
        alternating_sign[Real](a + b) * (x.pow(Nat.2 * a + e) * y.pow(Nat.2 * b + f))
    alternating_sign[Real](a + b) * x.pow(Nat.2 * a + e) * y.pow(Nat.2 * b + f) / (f1 * f2) =
        alternating_sign[Real](a + b) * (x.pow(Nat.2 * a + e) * y.pow(Nat.2 * b + f)) / (f1 * f2)
    alternating_sign[Real](a + b) * x.pow(Nat.2 * a + e) * y.pow(Nat.2 * b + f) / (f1 * f2) =
        alternating_sign[Real](a + b) * (from_nat[Real]((Nat.2 * a + e + Nat.2 * b + f).binom(Nat.2 * a + e)) * x.pow(Nat.2 * a + e) * y.pow(Nat.2 * b + f)) /
        Real.from_rat(Rat.from_nat((Nat.2 * a + e + Nat.2 * b + f).factorial))
    binomial_term[Real](x, y, Nat.2 * a + e + Nat.2 * b + f, Nat.2 * a + e) =
        from_nat[Real]((Nat.2 * a + e + Nat.2 * b + f).binom(Nat.2 * a + e)) * x.pow(Nat.2 * a + e) * y.pow(Nat.2 * a + e + Nat.2 * b + f - (Nat.2 * a + e))
    from_nat[Real]((Nat.2 * a + e + Nat.2 * b + f).binom(Nat.2 * a + e)) * x.pow(Nat.2 * a + e) * y.pow(Nat.2 * b + f) =
        from_nat[Real]((Nat.2 * a + e + Nat.2 * b + f).binom(Nat.2 * a + e)) * x.pow(Nat.2 * a + e) * y.pow(Nat.2 * a + e + Nat.2 * b + f - (Nat.2 * a + e))
    alternating_sign[Real](a + b) * (from_nat[Real]((Nat.2 * a + e + Nat.2 * b + f).binom(Nat.2 * a + e)) * x.pow(Nat.2 * a + e) * y.pow(Nat.2 * b + f)) /
        Real.from_rat(Rat.from_nat((Nat.2 * a + e + Nat.2 * b + f).factorial)) =
        alternating_sign[Real](a + b) * binomial_term[Real](x, y, Nat.2 * a + e + Nat.2 * b + f, Nat.2 * a + e) /
        Real.from_rat(Rat.from_nat((Nat.2 * a + e + Nat.2 * b + f).factorial))
    (alternating_sign[Real](a) * x.pow(Nat.2 * a + e) / f1) * (alternating_sign[Real](b) * y.pow(Nat.2 * b + f) / f2) =
        alternating_sign[Real](a + b) * binomial_term[Real](x, y, Nat.2 * a + e + Nat.2 * b + f, Nat.2 * a + e) /
        Real.from_rat(Rat.from_nat((Nat.2 * a + e + Nat.2 * b + f).factorial))
}

/// Doubling preserves addition over a difference: 2k + 2(n-k) = 2n.
theorem two_k_add_two_sub(n: Nat, k: Nat) {
    k <= n implies Nat.2 * k + Nat.2 * (n - k) = Nat.2 * n
} by {
    if k <= n {
        two_mul_sub(n, k)
        Nat.2 * (n - k) = Nat.2 * n - Nat.2 * k
        Nat.2 * k + Nat.2 * (n - k) = Nat.2 * k + (Nat.2 * n - Nat.2 * k)
        Nat.2 * k + (Nat.2 * n - Nat.2 * k) = (Nat.2 * n - Nat.2 * k) + Nat.2 * k
        lte_mul_both(Nat.2, k, n)
        Nat.2 * k <= Nat.2 * n
        add_sub(Nat.2 * n, Nat.2 * k)
        Nat.2 * n - Nat.2 * k + Nat.2 * k = Nat.2 * n
        Nat.2 * k + Nat.2 * (n - k) = Nat.2 * n
    }
}

/// The odd row of sine-times-cosine: 2k+1 + 2(n-k) = 2n+1.
theorem two_k_suc_add_two_sub(n: Nat, k: Nat) {
    k <= n implies Nat.2 * k + Nat.1 + Nat.2 * (n - k) = Nat.2 * n + Nat.1
} by {
    if k <= n {
        two_k_add_two_sub(n, k)
        Nat.2 * k + Nat.2 * (n - k) = Nat.2 * n
        Nat.2 * k + Nat.2 * (n - k) + Nat.1 = Nat.2 * n + Nat.1
        Nat.2 * k + Nat.1 + Nat.2 * (n - k) = Nat.2 * k + Nat.2 * (n - k) + Nat.1
        Nat.2 * k + Nat.1 + Nat.2 * (n - k) = Nat.2 * n + Nat.1
    }
}

/// The odd row of cosine-times-sine: 2k + 2(n-k) + 1 = 2n+1.
theorem two_k_add_two_sub_suc(n: Nat, k: Nat) {
    k <= n implies Nat.2 * k + Nat.2 * (n - k) + Nat.1 = Nat.2 * n + Nat.1
} by {
    if k <= n {
        two_k_add_two_sub(n, k)
        Nat.2 * k + Nat.2 * (n - k) = Nat.2 * n
        Nat.2 * k + Nat.2 * (n - k) + Nat.1 = Nat.2 * n + Nat.1
    }
}

/// The row of sine-times-sine: 2k+1 + 2(m-k) + 1 = 2m+2.
theorem two_k_suc_add_two_sub_suc(m: Nat, k: Nat) {
    k <= m implies Nat.2 * k + Nat.1 + Nat.2 * (m - k) + Nat.1 = Nat.2 * m + Nat.2
} by {
    if k <= m {
        two_k_add_two_sub(m, k)
        Nat.2 * k + Nat.2 * (m - k) = Nat.2 * m
        Nat.2 * k + Nat.2 * (m - k) + Nat.1 + Nat.1 = Nat.2 * m + Nat.1 + Nat.1
        Nat.2 * k + Nat.1 + Nat.2 * (m - k) + Nat.1 = Nat.2 * k + Nat.2 * (m - k) + Nat.1 + Nat.1
        Nat.2 * m + Nat.1 + Nat.1 = Nat.2 * m + Nat.2
        Nat.2 * k + Nat.1 + Nat.2 * (m - k) + Nat.1 = Nat.2 * m + Nat.2
    }
}

/// A sine term times a cosine term is the odd binomial term of the sine row.
theorem sin_cos_term_product(x: Real, y: Real, n: Nat, k: Nat) {
    k <= n implies
    sin_term(x, k) * cos_term(y, n - k) =
        alternating_sign[Real](n) * binomial_term[Real](x, y, Nat.2 * n + Nat.1, Nat.2 * k + Nat.1) /
        Real.from_rat(Rat.from_nat((Nat.2 * n + Nat.1).factorial))
} by {
    if k <= n {
        signed_terms_binom_product(x, y, k, n - k, Nat.1, Nat.0)
        (alternating_sign[Real](k) * x.pow(Nat.2 * k + Nat.1) / Real.from_rat(Rat.from_nat((Nat.2 * k + Nat.1).factorial))) *
        (alternating_sign[Real](n - k) * y.pow(Nat.2 * (n - k)) / Real.from_rat(Rat.from_nat((Nat.2 * (n - k)).factorial))) =
            alternating_sign[Real](k + (n - k)) * binomial_term[Real](x, y, Nat.2 * k + Nat.1 + Nat.2 * (n - k), Nat.2 * k + Nat.1) /
            Real.from_rat(Rat.from_nat((Nat.2 * k + Nat.1 + Nat.2 * (n - k)).factorial))
        sin_term(x, k) = alternating_sign[Real](k) * x.pow(Nat.2 * k + Nat.1) / Real.from_rat(Rat.from_nat((Nat.2 * k + Nat.1).factorial))
        cos_term(y, n - k) = alternating_sign[Real](n - k) * y.pow(Nat.2 * (n - k)) / Real.from_rat(Rat.from_nat((Nat.2 * (n - k)).factorial))
        sin_term(x, k) * cos_term(y, n - k) =
            alternating_sign[Real](k + (n - k)) * binomial_term[Real](x, y, Nat.2 * k + Nat.1 + Nat.2 * (n - k), Nat.2 * k + Nat.1) /
            Real.from_rat(Rat.from_nat((Nat.2 * k + Nat.1 + Nat.2 * (n - k)).factorial))
        add_sub(n, k)
        n - k + k = n
        k + (n - k) = n
        alternating_sign[Real](k + (n - k)) = alternating_sign[Real](n)
        two_k_suc_add_two_sub(n, k)
        Nat.2 * k + Nat.1 + Nat.2 * (n - k) = Nat.2 * n + Nat.1
        sin_term(x, k) * cos_term(y, n - k) =
            alternating_sign[Real](n) * binomial_term[Real](x, y, Nat.2 * n + Nat.1, Nat.2 * k + Nat.1) /
            Real.from_rat(Rat.from_nat((Nat.2 * n + Nat.1).factorial))
    }
}

/// A cosine term times a sine term is the even binomial term of the sine row.
theorem cos_sin_term_product(x: Real, y: Real, n: Nat, k: Nat) {
    k <= n implies
    cos_term(x, k) * sin_term(y, n - k) =
        alternating_sign[Real](n) * binomial_term[Real](x, y, Nat.2 * n + Nat.1, Nat.2 * k) /
        Real.from_rat(Rat.from_nat((Nat.2 * n + Nat.1).factorial))
} by {
    if k <= n {
        signed_terms_binom_product(x, y, k, n - k, Nat.0, Nat.1)
        (alternating_sign[Real](k) * x.pow(Nat.2 * k) / Real.from_rat(Rat.from_nat((Nat.2 * k).factorial))) *
        (alternating_sign[Real](n - k) * y.pow(Nat.2 * (n - k) + Nat.1) / Real.from_rat(Rat.from_nat((Nat.2 * (n - k) + Nat.1).factorial))) =
            alternating_sign[Real](k + (n - k)) * binomial_term[Real](x, y, Nat.2 * k + Nat.2 * (n - k) + Nat.1, Nat.2 * k) /
            Real.from_rat(Rat.from_nat((Nat.2 * k + Nat.2 * (n - k) + Nat.1).factorial))
        cos_term(x, k) = alternating_sign[Real](k) * x.pow(Nat.2 * k) / Real.from_rat(Rat.from_nat((Nat.2 * k).factorial))
        sin_term(y, n - k) = alternating_sign[Real](n - k) * y.pow(Nat.2 * (n - k) + Nat.1) / Real.from_rat(Rat.from_nat((Nat.2 * (n - k) + Nat.1).factorial))
        cos_term(x, k) * sin_term(y, n - k) =
            alternating_sign[Real](k + (n - k)) * binomial_term[Real](x, y, Nat.2 * k + Nat.2 * (n - k) + Nat.1, Nat.2 * k) /
            Real.from_rat(Rat.from_nat((Nat.2 * k + Nat.2 * (n - k) + Nat.1).factorial))
        add_sub(n, k)
        n - k + k = n
        k + (n - k) = n
        alternating_sign[Real](k + (n - k)) = alternating_sign[Real](n)
        two_k_add_two_sub_suc(n, k)
        Nat.2 * k + Nat.2 * (n - k) + Nat.1 = Nat.2 * n + Nat.1
        cos_term(x, k) * sin_term(y, n - k) =
            alternating_sign[Real](n) * binomial_term[Real](x, y, Nat.2 * n + Nat.1, Nat.2 * k) /
            Real.from_rat(Rat.from_nat((Nat.2 * n + Nat.1).factorial))
    }
}

/// A cosine term times a cosine term is the even binomial term of the cosine row.
theorem cos_cos_term_product(x: Real, y: Real, n: Nat, k: Nat) {
    k <= n implies
    cos_term(x, k) * cos_term(y, n - k) =
        alternating_sign[Real](n) * binomial_term[Real](x, y, Nat.2 * n, Nat.2 * k) /
        Real.from_rat(Rat.from_nat((Nat.2 * n).factorial))
} by {
    if k <= n {
        signed_terms_binom_product(x, y, k, n - k, Nat.0, Nat.0)
        (alternating_sign[Real](k) * x.pow(Nat.2 * k) / Real.from_rat(Rat.from_nat((Nat.2 * k).factorial))) *
        (alternating_sign[Real](n - k) * y.pow(Nat.2 * (n - k)) / Real.from_rat(Rat.from_nat((Nat.2 * (n - k)).factorial))) =
            alternating_sign[Real](k + (n - k)) * binomial_term[Real](x, y, Nat.2 * k + Nat.2 * (n - k), Nat.2 * k) /
            Real.from_rat(Rat.from_nat((Nat.2 * k + Nat.2 * (n - k)).factorial))
        cos_term(x, k) = alternating_sign[Real](k) * x.pow(Nat.2 * k) / Real.from_rat(Rat.from_nat((Nat.2 * k).factorial))
        cos_term(y, n - k) = alternating_sign[Real](n - k) * y.pow(Nat.2 * (n - k)) / Real.from_rat(Rat.from_nat((Nat.2 * (n - k)).factorial))
        cos_term(x, k) * cos_term(y, n - k) =
            alternating_sign[Real](k + (n - k)) * binomial_term[Real](x, y, Nat.2 * k + Nat.2 * (n - k), Nat.2 * k) /
            Real.from_rat(Rat.from_nat((Nat.2 * k + Nat.2 * (n - k)).factorial))
        add_sub(n, k)
        n - k + k = n
        k + (n - k) = n
        alternating_sign[Real](k + (n - k)) = alternating_sign[Real](n)
        two_k_add_two_sub(n, k)
        Nat.2 * k + Nat.2 * (n - k) = Nat.2 * n
        cos_term(x, k) * cos_term(y, n - k) =
            alternating_sign[Real](n) * binomial_term[Real](x, y, Nat.2 * n, Nat.2 * k) /
            Real.from_rat(Rat.from_nat((Nat.2 * n).factorial))
    }
}

/// A sine term times a sine term is the odd binomial term of the shifted row.
theorem sin_sin_term_product(x: Real, y: Real, m: Nat, k: Nat) {
    k <= m implies
    sin_term(x, k) * sin_term(y, m - k) =
        alternating_sign[Real](m) * binomial_term[Real](x, y, Nat.2 * m + Nat.2, Nat.2 * k + Nat.1) /
        Real.from_rat(Rat.from_nat((Nat.2 * m + Nat.2).factorial))
} by {
    if k <= m {
        signed_terms_binom_product(x, y, k, m - k, Nat.1, Nat.1)
        (alternating_sign[Real](k) * x.pow(Nat.2 * k + Nat.1) / Real.from_rat(Rat.from_nat((Nat.2 * k + Nat.1).factorial))) *
        (alternating_sign[Real](m - k) * y.pow(Nat.2 * (m - k) + Nat.1) / Real.from_rat(Rat.from_nat((Nat.2 * (m - k) + Nat.1).factorial))) =
            alternating_sign[Real](k + (m - k)) * binomial_term[Real](x, y, Nat.2 * k + Nat.1 + Nat.2 * (m - k) + Nat.1, Nat.2 * k + Nat.1) /
            Real.from_rat(Rat.from_nat((Nat.2 * k + Nat.1 + Nat.2 * (m - k) + Nat.1).factorial))
        sin_term(x, k) = alternating_sign[Real](k) * x.pow(Nat.2 * k + Nat.1) / Real.from_rat(Rat.from_nat((Nat.2 * k + Nat.1).factorial))
        sin_term(y, m - k) = alternating_sign[Real](m - k) * y.pow(Nat.2 * (m - k) + Nat.1) / Real.from_rat(Rat.from_nat((Nat.2 * (m - k) + Nat.1).factorial))
        sin_term(x, k) * sin_term(y, m - k) =
            alternating_sign[Real](k + (m - k)) * binomial_term[Real](x, y, Nat.2 * k + Nat.1 + Nat.2 * (m - k) + Nat.1, Nat.2 * k + Nat.1) /
            Real.from_rat(Rat.from_nat((Nat.2 * k + Nat.1 + Nat.2 * (m - k) + Nat.1).factorial))
        add_sub(m, k)
        m - k + k = m
        k + (m - k) = m
        alternating_sign[Real](k + (m - k)) = alternating_sign[Real](m)
        two_k_suc_add_two_sub_suc(m, k)
        Nat.2 * k + Nat.1 + Nat.2 * (m - k) + Nat.1 = Nat.2 * m + Nat.2
        sin_term(x, k) * sin_term(y, m - k) =
            alternating_sign[Real](m) * binomial_term[Real](x, y, Nat.2 * m + Nat.2, Nat.2 * k + Nat.1) /
            Real.from_rat(Rat.from_nat((Nat.2 * m + Nat.2).factorial))
    }
}

/// The odd-indexed binomial term of the sine row.
define sin_row_odd(x: Real, y: Real, n: Nat, k: Nat) -> Real {
    alternating_sign[Real](n) * binomial_term[Real](x, y, Nat.2 * n + Nat.1, Nat.2 * k + Nat.1) /
    Real.from_rat(Rat.from_nat((Nat.2 * n + Nat.1).factorial))
}

/// The even-indexed binomial term of the sine row.
define sin_row_even(x: Real, y: Real, n: Nat, k: Nat) -> Real {
    alternating_sign[Real](n) * binomial_term[Real](x, y, Nat.2 * n + Nat.1, Nat.2 * k) /
    Real.from_rat(Rat.from_nat((Nat.2 * n + Nat.1).factorial))
}

/// The pair sum of the even and odd binomial terms of the sine row.
define row_pair_sum(x: Real, y: Real, n: Nat, k: Nat) -> Real {
    binomial_term[Real](x, y, Nat.2 * n + Nat.1, Nat.2 * k) + binomial_term[Real](x, y, Nat.2 * n + Nat.1, Nat.2 * k + Nat.1)
}

/// The full coefficient of the sine addition formula.
define sin_row_sum(x: Real, y: Real, n: Nat, k: Nat) -> Real {
    alternating_sign[Real](n) * row_pair_sum(x, y, n, k) /
    Real.from_rat(Rat.from_nat((Nat.2 * n + Nat.1).factorial))
}

/// The Cauchy product of the sine series with the cosine series at index n,
/// plus the mirrored product, is the sine term of the sum.
theorem sin_add_cauchy(x: Real, y: Real, n: Nat) {
    cauchy_product(sin_term(x), cos_term(y), n) + cauchy_product(cos_term(x), sin_term(y), n) = sin_term(x + y, n)
} by {
    // Relate the first Cauchy product to the odd binomial row terms.
    forall(k: Nat) {
        if k < n.suc {
            cauchy_coefficient(sin_term(x), cos_term(y), n)(k) = sin_term(x, k) * cos_term(y, n - k)
            sin_cos_term_product(x, y, n, k)
            sin_term(x, k) * cos_term(y, n - k) =
                alternating_sign[Real](n) * binomial_term[Real](x, y, Nat.2 * n + Nat.1, Nat.2 * k + Nat.1) /
                Real.from_rat(Rat.from_nat((Nat.2 * n + Nat.1).factorial))
            sin_row_odd(x, y, n, k) =
                alternating_sign[Real](n) * binomial_term[Real](x, y, Nat.2 * n + Nat.1, Nat.2 * k + Nat.1) /
                Real.from_rat(Rat.from_nat((Nat.2 * n + Nat.1).factorial))
            cauchy_coefficient(sin_term(x), cos_term(y), n)(k) = sin_row_odd(x, y, n, k)
        }
    }
    partial_pointwise_eq(cauchy_coefficient(sin_term(x), cos_term(y), n), sin_row_odd(x, y, n), n.suc)
    partial(cauchy_coefficient(sin_term(x), cos_term(y), n), n.suc) = partial(sin_row_odd(x, y, n), n.suc)
    cauchy_product(sin_term(x), cos_term(y), n) = partial(cauchy_coefficient(sin_term(x), cos_term(y), n), n.suc)
    cauchy_product(sin_term(x), cos_term(y), n) = partial(sin_row_odd(x, y, n), n.suc)

    // Relate the second Cauchy product to the even binomial row terms.
    forall(k: Nat) {
        if k < n.suc {
            cauchy_coefficient(cos_term(x), sin_term(y), n)(k) = cos_term(x, k) * sin_term(y, n - k)
            cos_sin_term_product(x, y, n, k)
            cos_term(x, k) * sin_term(y, n - k) =
                alternating_sign[Real](n) * binomial_term[Real](x, y, Nat.2 * n + Nat.1, Nat.2 * k) /
                Real.from_rat(Rat.from_nat((Nat.2 * n + Nat.1).factorial))
            sin_row_even(x, y, n, k) =
                alternating_sign[Real](n) * binomial_term[Real](x, y, Nat.2 * n + Nat.1, Nat.2 * k) /
                Real.from_rat(Rat.from_nat((Nat.2 * n + Nat.1).factorial))
            cauchy_coefficient(cos_term(x), sin_term(y), n)(k) = sin_row_even(x, y, n, k)
        }
    }
    partial_pointwise_eq(cauchy_coefficient(cos_term(x), sin_term(y), n), sin_row_even(x, y, n), n.suc)
    partial(cauchy_coefficient(cos_term(x), sin_term(y), n), n.suc) = partial(sin_row_even(x, y, n), n.suc)
    cauchy_product(cos_term(x), sin_term(y), n) = partial(cauchy_coefficient(cos_term(x), sin_term(y), n), n.suc)
    cauchy_product(cos_term(x), sin_term(y), n) = partial(sin_row_even(x, y, n), n.suc)

    // The two partial sums combine into the pair-sum coefficient.
    cauchy_product(sin_term(x), cos_term(y), n) + cauchy_product(cos_term(x), sin_term(y), n) =
        partial(sin_row_odd(x, y, n), n.suc) + partial(sin_row_even(x, y, n), n.suc)
    partial_add(sin_row_odd(x, y, n), sin_row_even(x, y, n), n.suc)
    partial(sin_row_odd(x, y, n), n.suc) + partial(sin_row_even(x, y, n), n.suc) =
        partial(add_fn(sin_row_odd(x, y, n), sin_row_even(x, y, n)), n.suc)
    let f_row = Real.from_rat(Rat.from_nat((Nat.2 * n + Nat.1).factorial))
    f_row != Real.0
    forall(k: Nat) {
        sin_row_odd(x, y, n, k) =
            alternating_sign[Real](n) * binomial_term[Real](x, y, Nat.2 * n + Nat.1, Nat.2 * k + Nat.1) / f_row
        sin_row_even(x, y, n, k) =
            alternating_sign[Real](n) * binomial_term[Real](x, y, Nat.2 * n + Nat.1, Nat.2 * k) / f_row
        div_add_same_denom(alternating_sign[Real](n), binomial_term[Real](x, y, Nat.2 * n + Nat.1, Nat.2 * k + Nat.1), binomial_term[Real](x, y, Nat.2 * n + Nat.1, Nat.2 * k), f_row)
        alternating_sign[Real](n) * binomial_term[Real](x, y, Nat.2 * n + Nat.1, Nat.2 * k + Nat.1) / f_row +
        alternating_sign[Real](n) * binomial_term[Real](x, y, Nat.2 * n + Nat.1, Nat.2 * k) / f_row =
            alternating_sign[Real](n) * (binomial_term[Real](x, y, Nat.2 * n + Nat.1, Nat.2 * k + Nat.1) + binomial_term[Real](x, y, Nat.2 * n + Nat.1, Nat.2 * k)) / f_row
        sin_row_odd(x, y, n, k) + sin_row_even(x, y, n, k) =
            alternating_sign[Real](n) * (binomial_term[Real](x, y, Nat.2 * n + Nat.1, Nat.2 * k + Nat.1) + binomial_term[Real](x, y, Nat.2 * n + Nat.1, Nat.2 * k)) / f_row
        sin_row_sum(x, y, n, k) =
            alternating_sign[Real](n) * row_pair_sum(x, y, n, k) / f_row
        row_pair_sum(x, y, n, k) = binomial_term[Real](x, y, Nat.2 * n + Nat.1, Nat.2 * k) + binomial_term[Real](x, y, Nat.2 * n + Nat.1, Nat.2 * k + Nat.1)
        sin_row_sum(x, y, n, k) =
            alternating_sign[Real](n) * (binomial_term[Real](x, y, Nat.2 * n + Nat.1, Nat.2 * k + Nat.1) + binomial_term[Real](x, y, Nat.2 * n + Nat.1, Nat.2 * k)) / f_row
        sin_row_odd(x, y, n, k) + sin_row_even(x, y, n, k) = sin_row_sum(x, y, n, k)
    }
    partial_pointwise_eq(add_fn(sin_row_odd(x, y, n), sin_row_even(x, y, n)), sin_row_sum(x, y, n), n.suc)
    partial(add_fn(sin_row_odd(x, y, n), sin_row_even(x, y, n)), n.suc) = partial(sin_row_sum(x, y, n), n.suc)
    cauchy_product(sin_term(x), cos_term(y), n) + cauchy_product(cos_term(x), sin_term(y), n) =
        partial(sin_row_sum(x, y, n), n.suc)

    // The pair-sum coefficient is a scalar multiple of the full binomial row.
    forall(k: Nat) {
        sin_row_sum(x, y, n, k) =
            alternating_sign[Real](n) * row_pair_sum(x, y, n, k) / f_row
        div_fn(mul_fn(alternating_sign[Real](n), row_pair_sum(x, y, n)), f_row)(k) =
            alternating_sign[Real](n) * row_pair_sum(x, y, n, k) / f_row
        sin_row_sum(x, y, n, k) = div_fn(mul_fn(alternating_sign[Real](n), row_pair_sum(x, y, n)), f_row)(k)
    }
    sin_row_sum(x, y, n) = div_fn(mul_fn(alternating_sign[Real](n), row_pair_sum(x, y, n)), f_row)
    partial(sin_row_sum(x, y, n), n.suc) = partial(div_fn(mul_fn(alternating_sign[Real](n), row_pair_sum(x, y, n)), f_row), n.suc)
    f_row != Real.0
    partial_div_fn(mul_fn(alternating_sign[Real](n), row_pair_sum(x, y, n)), f_row, n.suc)
    partial(div_fn(mul_fn(alternating_sign[Real](n), row_pair_sum(x, y, n)), f_row), n.suc) =
        partial(mul_fn(alternating_sign[Real](n), row_pair_sum(x, y, n)), n.suc) / f_row
    partial_scalar_mul(alternating_sign[Real](n), row_pair_sum(x, y, n), n.suc)
    alternating_sign[Real](n) * partial(row_pair_sum(x, y, n), n.suc) =
        partial(mul_fn(alternating_sign[Real](n), row_pair_sum(x, y, n)), n.suc)
    partial(sin_row_sum(x, y, n), n.suc) =
        alternating_sign[Real](n) * partial(row_pair_sum(x, y, n), n.suc) / f_row

    // The pair sum of the row equals the full binomial row via the parity split.
    forall(k: Nat) {
        row_pair_sum(x, y, n, k) = binomial_term[Real](x, y, Nat.2 * n + Nat.1, Nat.2 * k) + binomial_term[Real](x, y, Nat.2 * n + Nat.1, Nat.2 * k + Nat.1)
        even_subseq(binomial_term[Real](x, y, Nat.2 * n + Nat.1), k) = binomial_term[Real](x, y, Nat.2 * n + Nat.1, Nat.2 * k)
        odd_subseq(binomial_term[Real](x, y, Nat.2 * n + Nat.1), k) = binomial_term[Real](x, y, Nat.2 * n + Nat.1, Nat.2 * k + Nat.1)
        add_fn(even_subseq(binomial_term[Real](x, y, Nat.2 * n + Nat.1)), odd_subseq(binomial_term[Real](x, y, Nat.2 * n + Nat.1)), k) =
            even_subseq(binomial_term[Real](x, y, Nat.2 * n + Nat.1), k) + odd_subseq(binomial_term[Real](x, y, Nat.2 * n + Nat.1), k)
        row_pair_sum(x, y, n, k) = add_fn(even_subseq(binomial_term[Real](x, y, Nat.2 * n + Nat.1)), odd_subseq(binomial_term[Real](x, y, Nat.2 * n + Nat.1)), k)
    }
    row_pair_sum(x, y, n) = add_fn(even_subseq(binomial_term[Real](x, y, Nat.2 * n + Nat.1)), odd_subseq(binomial_term[Real](x, y, Nat.2 * n + Nat.1)))
    partial(row_pair_sum(x, y, n), n.suc) =
        partial(add_fn(even_subseq(binomial_term[Real](x, y, Nat.2 * n + Nat.1)), odd_subseq(binomial_term[Real](x, y, Nat.2 * n + Nat.1))), n.suc)
    partial_add(even_subseq(binomial_term[Real](x, y, Nat.2 * n + Nat.1)), odd_subseq(binomial_term[Real](x, y, Nat.2 * n + Nat.1)), n.suc)
    partial(even_subseq(binomial_term[Real](x, y, Nat.2 * n + Nat.1)), n.suc) + partial(odd_subseq(binomial_term[Real](x, y, Nat.2 * n + Nat.1)), n.suc) =
        partial(add_fn(even_subseq(binomial_term[Real](x, y, Nat.2 * n + Nat.1)), odd_subseq(binomial_term[Real](x, y, Nat.2 * n + Nat.1))), n.suc)
    partial(row_pair_sum(x, y, n), n.suc) =
        partial(even_subseq(binomial_term[Real](x, y, Nat.2 * n + Nat.1)), n.suc) + partial(odd_subseq(binomial_term[Real](x, y, Nat.2 * n + Nat.1)), n.suc)
    partial_split_even(binomial_term[Real](x, y, Nat.2 * n + Nat.1), n)
    partial(binomial_term[Real](x, y, Nat.2 * n + Nat.1), Nat.2 * n.suc) =
        partial(even_subseq(binomial_term[Real](x, y, Nat.2 * n + Nat.1)), n.suc) + partial(odd_subseq(binomial_term[Real](x, y, Nat.2 * n + Nat.1)), n.suc)
    partial(row_pair_sum(x, y, n), n.suc) = partial(binomial_term[Real](x, y, Nat.2 * n + Nat.1), Nat.2 * n.suc)

    // The full binomial row is (x+y)^(2n+1).
    binomial[Real](x, y, Nat.2 * n + Nat.1)
    (x + y).pow(Nat.2 * n + Nat.1) = partial(binomial_term[Real](x, y, Nat.2 * n + Nat.1), (Nat.2 * n + Nat.1).suc)
    two_mul_suc(n)
    Nat.2 * n.suc = (Nat.2 * n).suc.suc
    (Nat.2 * n).suc.suc = (Nat.2 * n + Nat.1).suc
    Nat.2 * n.suc = (Nat.2 * n + Nat.1).suc
    partial(binomial_term[Real](x, y, Nat.2 * n + Nat.1), Nat.2 * n.suc) = (x + y).pow(Nat.2 * n + Nat.1)
    partial(row_pair_sum(x, y, n), n.suc) = (x + y).pow(Nat.2 * n + Nat.1)

    // Assemble the sine term of the sum.
    partial(sin_row_sum(x, y, n), n.suc) =
        alternating_sign[Real](n) * (x + y).pow(Nat.2 * n + Nat.1) / f_row
    sin_term(x + y, n) =
        alternating_sign[Real](n) * (x + y).pow(Nat.2 * n + Nat.1) /
        Real.from_rat(Rat.from_nat((Nat.2 * n + Nat.1).factorial))
    partial(sin_row_sum(x, y, n), n.suc) = sin_term(x + y, n)
    cauchy_product(sin_term(x), cos_term(y), n) + cauchy_product(cos_term(x), sin_term(y), n) = sin_term(x + y, n)
}

/// Subtracting two fractions with the same denominator and shared factors.
theorem div_sub_same_denom(a: Real, b: Real, x: Real, y: Real, f: Real) {
    f != Real.0 implies a * x / f - b * y / f = (a * x - b * y) / f
} by {
    if f != Real.0 {
        a * x / f = a * x * f.inverse
        a * x / f - b * y / f = a * x * f.inverse - b * y / f
        b * y / f = b * y * f.inverse
        a * x / f - b * y / f = a * x * f.inverse - b * y * f.inverse
        (a * x - b * y) / f = (a * x - b * y) * f.inverse
        a * x * f.inverse - b * y * f.inverse = (a * x - b * y) * f.inverse
        a * x / f - b * y / f = (a * x - b * y) / f
    }
}

/// Doubling the predecessor and adding two recovers the double: 2(n-1)+2 = 2n.
theorem two_pred_add_two(n: Nat) {
    n >= Nat.1 implies Nat.2 * (n - Nat.1) + Nat.2 = Nat.2 * n
} by {
    if n >= Nat.1 {
        add_sub(n, Nat.1)
        n - Nat.1 + Nat.1 = n
        Nat.2 * (n - Nat.1 + Nat.1) = Nat.2 * n
        two_mul_suc(n - Nat.1)
        Nat.2 * (n - Nat.1).suc = (Nat.2 * (n - Nat.1)).suc.suc
        Nat.2 * (n - Nat.1) + Nat.2 = (Nat.2 * (n - Nat.1)).suc.suc
        (n - Nat.1) + Nat.1 = (n - Nat.1).suc
        Nat.2 * ((n - Nat.1) + Nat.1) = Nat.2 * (n - Nat.1).suc
        Nat.2 * ((n - Nat.1) + Nat.1) = Nat.2 * (n - Nat.1) + Nat.2
        Nat.2 * (n - Nat.1) + Nat.2 = Nat.2 * n
    }
}

/// The even-indexed binomial term of the cosine row.
define cos_row_even(x: Real, y: Real, n: Nat, k: Nat) -> Real {
    alternating_sign[Real](n) * binomial_term[Real](x, y, Nat.2 * n, Nat.2 * k) /
    Real.from_rat(Rat.from_nat((Nat.2 * n).factorial))
}

/// The odd-indexed binomial term of the shifted sine row.
define sin_row_odd_shift(x: Real, y: Real, m: Nat, k: Nat) -> Real {
    alternating_sign[Real](m) * binomial_term[Real](x, y, Nat.2 * m + Nat.2, Nat.2 * k + Nat.1) /
    Real.from_rat(Rat.from_nat((Nat.2 * m + Nat.2).factorial))
}

/// The predecessor of a positive number succeeds to the number: (n-1)+1 = n.
theorem pred_suc(n: Nat) {
    n >= Nat.1 implies (n - Nat.1).suc = n
} by {
    if n >= Nat.1 {
        add_sub(n, Nat.1)
        n - Nat.1 + Nat.1 = n
        add_one_right(n - Nat.1)
        (n - Nat.1) + Nat.1 = (n - Nat.1).suc
        (n - Nat.1).suc = n
    }
}

/// The Cauchy product of the cosine series with itself at index n, minus the
/// sine-sine product at index n-1, is the cosine term of the sum.
theorem cos_add_cauchy(x: Real, y: Real, n: Nat) {
    n >= Nat.1 implies
    cauchy_product(cos_term(x), cos_term(y), n) - cauchy_product(sin_term(x), sin_term(y), n - Nat.1) = cos_term(x + y, n)
} by {
    if n >= Nat.1 {
        // Relate the cosine-cosine product to the even binomial row terms.
        forall(k: Nat) {
            if k < n.suc {
                cauchy_coefficient(cos_term(x), cos_term(y), n)(k) = cos_term(x, k) * cos_term(y, n - k)
                cos_cos_term_product(x, y, n, k)
                cos_term(x, k) * cos_term(y, n - k) =
                    alternating_sign[Real](n) * binomial_term[Real](x, y, Nat.2 * n, Nat.2 * k) /
                    Real.from_rat(Rat.from_nat((Nat.2 * n).factorial))
                cos_row_even(x, y, n, k) =
                    alternating_sign[Real](n) * binomial_term[Real](x, y, Nat.2 * n, Nat.2 * k) /
                    Real.from_rat(Rat.from_nat((Nat.2 * n).factorial))
                cauchy_coefficient(cos_term(x), cos_term(y), n)(k) = cos_row_even(x, y, n, k)
            }
        }
        partial_pointwise_eq(cauchy_coefficient(cos_term(x), cos_term(y), n), cos_row_even(x, y, n), n.suc)
        partial(cauchy_coefficient(cos_term(x), cos_term(y), n), n.suc) = partial(cos_row_even(x, y, n), n.suc)
        cauchy_product(cos_term(x), cos_term(y), n) = partial(cauchy_coefficient(cos_term(x), cos_term(y), n), n.suc)
        cauchy_product(cos_term(x), cos_term(y), n) = partial(cos_row_even(x, y, n), n.suc)

        // Relate the sine-sine product at n-1 to the odd binomial row terms.
        forall(k: Nat) {
            if k < n {
                cauchy_coefficient(sin_term(x), sin_term(y), n - Nat.1)(k) = sin_term(x, k) * sin_term(y, n - Nat.1 - k)
                pred_suc(n)
                (n - Nat.1).suc = n
                k < (n - Nat.1).suc
                k <= n - Nat.1
                sin_sin_term_product(x, y, n - Nat.1, k)
                sin_term(x, k) * sin_term(y, n - Nat.1 - k) =
                    alternating_sign[Real](n - Nat.1) * binomial_term[Real](x, y, Nat.2 * (n - Nat.1) + Nat.2, Nat.2 * k + Nat.1) /
                    Real.from_rat(Rat.from_nat((Nat.2 * (n - Nat.1) + Nat.2).factorial))
                two_pred_add_two(n)
                Nat.2 * (n - Nat.1) + Nat.2 = Nat.2 * n
                sin_term(x, k) * sin_term(y, n - Nat.1 - k) =
                    alternating_sign[Real](n - Nat.1) * binomial_term[Real](x, y, Nat.2 * n, Nat.2 * k + Nat.1) /
                    Real.from_rat(Rat.from_nat((Nat.2 * n).factorial))
                sin_row_odd_shift(x, y, n - Nat.1, k) =
                    alternating_sign[Real](n - Nat.1) * binomial_term[Real](x, y, Nat.2 * (n - Nat.1) + Nat.2, Nat.2 * k + Nat.1) /
                    Real.from_rat(Rat.from_nat((Nat.2 * (n - Nat.1) + Nat.2).factorial))
                sin_row_odd_shift(x, y, n - Nat.1, k) =
                    alternating_sign[Real](n - Nat.1) * binomial_term[Real](x, y, Nat.2 * n, Nat.2 * k + Nat.1) /
                    Real.from_rat(Rat.from_nat((Nat.2 * n).factorial))
                cauchy_coefficient(sin_term(x), sin_term(y), n - Nat.1)(k) = sin_row_odd_shift(x, y, n - Nat.1, k)
            }
        }
        partial_pointwise_eq(cauchy_coefficient(sin_term(x), sin_term(y), n - Nat.1), sin_row_odd_shift(x, y, n - Nat.1), n)
        partial(cauchy_coefficient(sin_term(x), sin_term(y), n - Nat.1), n) = partial(sin_row_odd_shift(x, y, n - Nat.1), n)
        cauchy_product(sin_term(x), sin_term(y), n - Nat.1) = partial(cauchy_coefficient(sin_term(x), sin_term(y), n - Nat.1), (n - Nat.1).suc)
        pred_suc(n)
        (n - Nat.1).suc = n
        cauchy_product(sin_term(x), sin_term(y), n - Nat.1) = partial(cauchy_coefficient(sin_term(x), sin_term(y), n - Nat.1), n)
        cauchy_product(sin_term(x), sin_term(y), n - Nat.1) = partial(sin_row_odd_shift(x, y, n - Nat.1), n)

        // Form the difference of the two partial sums.
        cauchy_product(cos_term(x), cos_term(y), n) - cauchy_product(sin_term(x), sin_term(y), n - Nat.1) =
            partial(cos_row_even(x, y, n), n.suc) - partial(sin_row_odd_shift(x, y, n - Nat.1), n)

        // Factor out the alternating sign and the factorial from each part.
        let f_cos = Real.from_rat(Rat.from_nat((Nat.2 * n).factorial))
        f_cos != Real.0
        forall(k: Nat) {
            cos_row_even(x, y, n, k) =
                alternating_sign[Real](n) * even_subseq(binomial_term[Real](x, y, Nat.2 * n), k) / f_cos
            div_fn(mul_fn(alternating_sign[Real](n), even_subseq(binomial_term[Real](x, y, Nat.2 * n))), f_cos)(k) =
                alternating_sign[Real](n) * even_subseq(binomial_term[Real](x, y, Nat.2 * n), k) / f_cos
            cos_row_even(x, y, n, k) = div_fn(mul_fn(alternating_sign[Real](n), even_subseq(binomial_term[Real](x, y, Nat.2 * n))), f_cos)(k)
        }
        cos_row_even(x, y, n) = div_fn(mul_fn(alternating_sign[Real](n), even_subseq(binomial_term[Real](x, y, Nat.2 * n))), f_cos)
        partial(cos_row_even(x, y, n), n.suc) = partial(div_fn(mul_fn(alternating_sign[Real](n), even_subseq(binomial_term[Real](x, y, Nat.2 * n))), f_cos), n.suc)
        partial_div_fn(mul_fn(alternating_sign[Real](n), even_subseq(binomial_term[Real](x, y, Nat.2 * n))), f_cos, n.suc)
        partial(div_fn(mul_fn(alternating_sign[Real](n), even_subseq(binomial_term[Real](x, y, Nat.2 * n))), f_cos), n.suc) =
            partial(mul_fn(alternating_sign[Real](n), even_subseq(binomial_term[Real](x, y, Nat.2 * n))), n.suc) / f_cos
        partial_scalar_mul(alternating_sign[Real](n), even_subseq(binomial_term[Real](x, y, Nat.2 * n)), n.suc)
        alternating_sign[Real](n) * partial(even_subseq(binomial_term[Real](x, y, Nat.2 * n)), n.suc) =
            partial(mul_fn(alternating_sign[Real](n), even_subseq(binomial_term[Real](x, y, Nat.2 * n))), n.suc)
        partial(cos_row_even(x, y, n), n.suc) =
            alternating_sign[Real](n) * partial(even_subseq(binomial_term[Real](x, y, Nat.2 * n)), n.suc) / f_cos

        forall(k: Nat) {
            sin_row_odd_shift(x, y, n - Nat.1, k) =
                alternating_sign[Real](n - Nat.1) * binomial_term[Real](x, y, Nat.2 * (n - Nat.1) + Nat.2, Nat.2 * k + Nat.1) /
                Real.from_rat(Rat.from_nat((Nat.2 * (n - Nat.1) + Nat.2).factorial))
            two_pred_add_two(n)
            Nat.2 * (n - Nat.1) + Nat.2 = Nat.2 * n
            sin_row_odd_shift(x, y, n - Nat.1, k) =
                alternating_sign[Real](n - Nat.1) * binomial_term[Real](x, y, Nat.2 * n, Nat.2 * k + Nat.1) / f_cos
            odd_subseq(binomial_term[Real](x, y, Nat.2 * n), k) =
                binomial_term[Real](x, y, Nat.2 * n, Nat.2 * k + Nat.1)
            sin_row_odd_shift(x, y, n - Nat.1, k) =
                alternating_sign[Real](n - Nat.1) * odd_subseq(binomial_term[Real](x, y, Nat.2 * n), k) / f_cos
            div_fn(mul_fn(alternating_sign[Real](n - Nat.1), odd_subseq(binomial_term[Real](x, y, Nat.2 * n))), f_cos)(k) =
                alternating_sign[Real](n - Nat.1) * odd_subseq(binomial_term[Real](x, y, Nat.2 * n), k) / f_cos
            sin_row_odd_shift(x, y, n - Nat.1, k) = div_fn(mul_fn(alternating_sign[Real](n - Nat.1), odd_subseq(binomial_term[Real](x, y, Nat.2 * n))), f_cos)(k)
        }
        sin_row_odd_shift(x, y, n - Nat.1) = div_fn(mul_fn(alternating_sign[Real](n - Nat.1), odd_subseq(binomial_term[Real](x, y, Nat.2 * n))), f_cos)
        partial(sin_row_odd_shift(x, y, n - Nat.1), n) = partial(div_fn(mul_fn(alternating_sign[Real](n - Nat.1), odd_subseq(binomial_term[Real](x, y, Nat.2 * n))), f_cos), n)
        partial_div_fn(mul_fn(alternating_sign[Real](n - Nat.1), odd_subseq(binomial_term[Real](x, y, Nat.2 * n))), f_cos, n)
        partial(div_fn(mul_fn(alternating_sign[Real](n - Nat.1), odd_subseq(binomial_term[Real](x, y, Nat.2 * n))), f_cos), n) =
            partial(mul_fn(alternating_sign[Real](n - Nat.1), odd_subseq(binomial_term[Real](x, y, Nat.2 * n))), n) / f_cos
        partial_scalar_mul(alternating_sign[Real](n - Nat.1), odd_subseq(binomial_term[Real](x, y, Nat.2 * n)), n)
        alternating_sign[Real](n - Nat.1) * partial(odd_subseq(binomial_term[Real](x, y, Nat.2 * n)), n) =
            partial(mul_fn(alternating_sign[Real](n - Nat.1), odd_subseq(binomial_term[Real](x, y, Nat.2 * n))), n)
        partial(sin_row_odd_shift(x, y, n - Nat.1), n) =
            alternating_sign[Real](n - Nat.1) * partial(odd_subseq(binomial_term[Real](x, y, Nat.2 * n)), n) / f_cos

        // The difference combines the two row parts.
        cauchy_product(cos_term(x), cos_term(y), n) - cauchy_product(sin_term(x), sin_term(y), n - Nat.1) =
            alternating_sign[Real](n) * partial(even_subseq(binomial_term[Real](x, y, Nat.2 * n)), n.suc) / f_cos -
            alternating_sign[Real](n - Nat.1) * partial(odd_subseq(binomial_term[Real](x, y, Nat.2 * n)), n) / f_cos
        div_sub_same_denom(alternating_sign[Real](n), alternating_sign[Real](n - Nat.1),
            partial(even_subseq(binomial_term[Real](x, y, Nat.2 * n)), n.suc), partial(odd_subseq(binomial_term[Real](x, y, Nat.2 * n)), n), f_cos)
        alternating_sign[Real](n) * partial(even_subseq(binomial_term[Real](x, y, Nat.2 * n)), n.suc) / f_cos -
        alternating_sign[Real](n - Nat.1) * partial(odd_subseq(binomial_term[Real](x, y, Nat.2 * n)), n) / f_cos =
            (alternating_sign[Real](n) * partial(even_subseq(binomial_term[Real](x, y, Nat.2 * n)), n.suc) -
             alternating_sign[Real](n - Nat.1) * partial(odd_subseq(binomial_term[Real](x, y, Nat.2 * n)), n)) / f_cos
        alternating_sign_suc[Real](n - Nat.1)
        alternating_sign[Real]((n - Nat.1).suc) = -alternating_sign[Real](n - Nat.1)
        pred_suc(n)
        (n - Nat.1).suc = n
        alternating_sign[Real](n) = -alternating_sign[Real](n - Nat.1)
        alternating_sign[Real](n - Nat.1) = -alternating_sign[Real](n)
        alternating_sign[Real](n - Nat.1) * partial(odd_subseq(binomial_term[Real](x, y, Nat.2 * n)), n) =
            -(alternating_sign[Real](n) * partial(odd_subseq(binomial_term[Real](x, y, Nat.2 * n)), n))
        alternating_sign[Real](n) * partial(even_subseq(binomial_term[Real](x, y, Nat.2 * n)), n.suc) -
        alternating_sign[Real](n - Nat.1) * partial(odd_subseq(binomial_term[Real](x, y, Nat.2 * n)), n) =
            alternating_sign[Real](n) * partial(even_subseq(binomial_term[Real](x, y, Nat.2 * n)), n.suc) -
            (-(alternating_sign[Real](n) * partial(odd_subseq(binomial_term[Real](x, y, Nat.2 * n)), n)))
        alternating_sign[Real](n) * partial(even_subseq(binomial_term[Real](x, y, Nat.2 * n)), n.suc) -
        (-(alternating_sign[Real](n) * partial(odd_subseq(binomial_term[Real](x, y, Nat.2 * n)), n))) =
            alternating_sign[Real](n) * partial(even_subseq(binomial_term[Real](x, y, Nat.2 * n)), n.suc) +
            alternating_sign[Real](n) * partial(odd_subseq(binomial_term[Real](x, y, Nat.2 * n)), n)
        (alternating_sign[Real](n) * partial(even_subseq(binomial_term[Real](x, y, Nat.2 * n)), n.suc) -
         alternating_sign[Real](n - Nat.1) * partial(odd_subseq(binomial_term[Real](x, y, Nat.2 * n)), n)) / f_cos =
            (alternating_sign[Real](n) * partial(even_subseq(binomial_term[Real](x, y, Nat.2 * n)), n.suc) +
             alternating_sign[Real](n) * partial(odd_subseq(binomial_term[Real](x, y, Nat.2 * n)), n)) / f_cos
        (alternating_sign[Real](n) * partial(even_subseq(binomial_term[Real](x, y, Nat.2 * n)), n.suc) +
         alternating_sign[Real](n) * partial(odd_subseq(binomial_term[Real](x, y, Nat.2 * n)), n)) / f_cos =
            alternating_sign[Real](n) * (partial(even_subseq(binomial_term[Real](x, y, Nat.2 * n)), n.suc) +
            partial(odd_subseq(binomial_term[Real](x, y, Nat.2 * n)), n)) / f_cos
        cauchy_product(cos_term(x), cos_term(y), n) - cauchy_product(sin_term(x), sin_term(y), n - Nat.1) =
            alternating_sign[Real](n) * (partial(even_subseq(binomial_term[Real](x, y, Nat.2 * n)), n.suc) +
            partial(odd_subseq(binomial_term[Real](x, y, Nat.2 * n)), n)) / f_cos

        // The parity split combines the two row parts into the full row.
        partial_split_odd(binomial_term[Real](x, y, Nat.2 * n), n)
        partial(binomial_term[Real](x, y, Nat.2 * n), Nat.2 * n + Nat.1) =
            partial(even_subseq(binomial_term[Real](x, y, Nat.2 * n)), n.suc) + partial(odd_subseq(binomial_term[Real](x, y, Nat.2 * n)), n)
        cauchy_product(cos_term(x), cos_term(y), n) - cauchy_product(sin_term(x), sin_term(y), n - Nat.1) =
            alternating_sign[Real](n) * partial(binomial_term[Real](x, y, Nat.2 * n), Nat.2 * n + Nat.1) / f_cos

        // The full binomial row is (x+y)^(2n).
        binomial[Real](x, y, Nat.2 * n)
        (x + y).pow(Nat.2 * n) = partial(binomial_term[Real](x, y, Nat.2 * n), (Nat.2 * n).suc)
        (Nat.2 * n).suc = Nat.2 * n + Nat.1
        partial(binomial_term[Real](x, y, Nat.2 * n), Nat.2 * n + Nat.1) = (x + y).pow(Nat.2 * n)
        cauchy_product(cos_term(x), cos_term(y), n) - cauchy_product(sin_term(x), sin_term(y), n - Nat.1) =
            alternating_sign[Real](n) * (x + y).pow(Nat.2 * n) / f_cos

        // This is the cosine term of the sum.
        cos_term(x + y, n) =
            alternating_sign[Real](n) * (x + y).pow(Nat.2 * n) /
            Real.from_rat(Rat.from_nat((Nat.2 * n).factorial))
        cauchy_product(cos_term(x), cos_term(y), n) - cauchy_product(sin_term(x), sin_term(y), n - Nat.1) = cos_term(x + y, n)
    }
}

/// The partial sums of the cosine series of the sum equal the difference of
/// the cosine-cosine and sine-sine Cauchy partial sums.
theorem cos_add_cauchy_seq(x: Real, y: Real, n: Nat) {
    partial(cos_term(x + y), n) =
        partial(cauchy_seq(cos_term(x), cos_term(y)), n) -
        partial(cauchy_seq(sin_term(x), sin_term(y)), n - Nat.1)
} by {
    define p(k: Nat) -> Bool {
        partial(cos_term(x + y), k) =
            partial(cauchy_seq(cos_term(x), cos_term(y)), k) -
            partial(cauchy_seq(sin_term(x), sin_term(y)), k - Nat.1)
    }

    // Base case at zero.
    partial_zero(cos_term(x + y))
    partial(cos_term(x + y), Nat.0) = Real.0
    partial_zero(cauchy_seq(cos_term(x), cos_term(y)))
    partial(cauchy_seq(cos_term(x), cos_term(y)), Nat.0) = Real.0
    partial_zero(cauchy_seq(sin_term(x), sin_term(y)))
    partial(cauchy_seq(sin_term(x), sin_term(y)), Nat.0) = Real.0
    Nat.0 - Nat.1 = Nat.0
    partial(cauchy_seq(sin_term(x), sin_term(y)), Nat.0 - Nat.1) = Real.0
    p(Nat.0)

    // Base case at one.
    partial_one(cos_term(x + y))
    partial(cos_term(x + y), Nat.1) = cos_term(x + y, Nat.0)
    cos_term(x + y, Nat.0) = alternating_sign[Real](Nat.0) * (x + y).pow(Nat.2 * Nat.0) / Real.from_rat(Rat.from_nat((Nat.2 * Nat.0).factorial))
    alternating_sign_zero[Real]
    alternating_sign[Real](Nat.0) = Real.1
    Nat.2 * Nat.0 = Nat.0
    (x + y).pow(Nat.0) = Real.1
    Nat.0.factorial = Nat.1
    Real.from_rat(Rat.from_nat(Nat.1)) = Real.1
    cos_term(x + y, Nat.0) = Real.1
    partial(cos_term(x + y), Nat.1) = Real.1
    cauchy_product_zero(cos_term(x), cos_term(y))
    cauchy_product(cos_term(x), cos_term(y), Nat.0) = cos_term(x, Nat.0) * cos_term(y, Nat.0)
    cos_term(x, Nat.0) = alternating_sign[Real](Nat.0) * x.pow(Nat.2 * Nat.0) / Real.from_rat(Rat.from_nat((Nat.2 * Nat.0).factorial))
    alternating_sign[Real](Nat.0) = Real.1
    Nat.2 * Nat.0 = Nat.0
    x.pow(Nat.0) = Real.1
    Nat.0.factorial = Nat.1
    Real.from_rat(Rat.from_nat(Nat.1)) = Real.1
    cos_term(x, Nat.0) = Real.1
    cos_term(y, Nat.0) = alternating_sign[Real](Nat.0) * y.pow(Nat.2 * Nat.0) / Real.from_rat(Rat.from_nat((Nat.2 * Nat.0).factorial))
    alternating_sign[Real](Nat.0) = Real.1
    Nat.2 * Nat.0 = Nat.0
    y.pow(Nat.0) = Real.1
    Nat.0.factorial = Nat.1
    Real.from_rat(Rat.from_nat(Nat.1)) = Real.1
    cos_term(y, Nat.0) = Real.1
    cauchy_product(cos_term(x), cos_term(y), Nat.0) = Real.1
    partial_one(cauchy_seq(cos_term(x), cos_term(y)))
    partial(cauchy_seq(cos_term(x), cos_term(y)), Nat.1) = cauchy_seq(cos_term(x), cos_term(y))(Nat.0)
    cauchy_seq(cos_term(x), cos_term(y))(Nat.0) = cauchy_product(cos_term(x), cos_term(y), Nat.0)
    partial(cauchy_seq(cos_term(x), cos_term(y)), Nat.1) = Real.1
    Nat.1 - Nat.1 = Nat.0
    partial(cauchy_seq(sin_term(x), sin_term(y)), Nat.1 - Nat.1) = Real.0
    p(Nat.1)

    // Inductive step.
    forall(k: Nat) {
        if p(k) {
            if k = Nat.0 {
                p(k.suc)
            } else {
                k >= Nat.1
                p(k)
                partial(cos_term(x + y), k) =
                    partial(cauchy_seq(cos_term(x), cos_term(y)), k) -
                    partial(cauchy_seq(sin_term(x), sin_term(y)), k - Nat.1)
                partial_split_last(cos_term(x + y), k)
                partial(cos_term(x + y), k.suc) = partial(cos_term(x + y), k) + cos_term(x + y, k)
                partial_split_last(cauchy_seq(cos_term(x), cos_term(y)), k)
                partial(cauchy_seq(cos_term(x), cos_term(y)), k.suc) =
                    partial(cauchy_seq(cos_term(x), cos_term(y)), k) + cauchy_seq(cos_term(x), cos_term(y))(k)
                cauchy_seq(cos_term(x), cos_term(y))(k) = cauchy_product(cos_term(x), cos_term(y), k)
                pred_suc(k)
                (k - Nat.1).suc = k
                partial_split_last(cauchy_seq(sin_term(x), sin_term(y)), k - Nat.1)
                partial(cauchy_seq(sin_term(x), sin_term(y)), (k - Nat.1).suc) =
                    partial(cauchy_seq(sin_term(x), sin_term(y)), k - Nat.1) + cauchy_seq(sin_term(x), sin_term(y))(k - Nat.1)
                partial(cauchy_seq(sin_term(x), sin_term(y)), k) =
                    partial(cauchy_seq(sin_term(x), sin_term(y)), k - Nat.1) + cauchy_seq(sin_term(x), sin_term(y))(k - Nat.1)
                cauchy_seq(sin_term(x), sin_term(y))(k - Nat.1) = cauchy_product(sin_term(x), sin_term(y), k - Nat.1)
                cos_add_cauchy(x, y, k)
                cauchy_product(cos_term(x), cos_term(y), k) - cauchy_product(sin_term(x), sin_term(y), k - Nat.1) =
                    cos_term(x + y, k)
                partial(cos_term(x + y), k.suc) =
                    (partial(cauchy_seq(cos_term(x), cos_term(y)), k) - partial(cauchy_seq(sin_term(x), sin_term(y)), k - Nat.1)) +
                    (cauchy_product(cos_term(x), cos_term(y), k) - cauchy_product(sin_term(x), sin_term(y), k - Nat.1))
                sub_add_sub(partial(cauchy_seq(cos_term(x), cos_term(y)), k), partial(cauchy_seq(sin_term(x), sin_term(y)), k - Nat.1),
                    cauchy_product(cos_term(x), cos_term(y), k), cauchy_product(sin_term(x), sin_term(y), k - Nat.1))
                (partial(cauchy_seq(cos_term(x), cos_term(y)), k) - partial(cauchy_seq(sin_term(x), sin_term(y)), k - Nat.1)) +
                (cauchy_product(cos_term(x), cos_term(y), k) - cauchy_product(sin_term(x), sin_term(y), k - Nat.1)) =
                    (partial(cauchy_seq(cos_term(x), cos_term(y)), k) + cauchy_product(cos_term(x), cos_term(y), k)) -
                    (partial(cauchy_seq(sin_term(x), sin_term(y)), k - Nat.1) + cauchy_product(sin_term(x), sin_term(y), k - Nat.1))
                partial(cos_term(x + y), k.suc) =
                    (partial(cauchy_seq(cos_term(x), cos_term(y)), k) + cauchy_product(cos_term(x), cos_term(y), k)) -
                    (partial(cauchy_seq(sin_term(x), sin_term(y)), k - Nat.1) + cauchy_product(sin_term(x), sin_term(y), k - Nat.1))
                partial(cos_term(x + y), k.suc) =
                    partial(cauchy_seq(cos_term(x), cos_term(y)), k.suc) -
                    (partial(cauchy_seq(sin_term(x), sin_term(y)), k - Nat.1) + cauchy_product(sin_term(x), sin_term(y), k - Nat.1))
                partial(cos_term(x + y), k.suc) =
                    partial(cauchy_seq(cos_term(x), cos_term(y)), k.suc) -
                    partial(cauchy_seq(sin_term(x), sin_term(y)), k)
                suc_sub_one(k.suc)
                k.suc - Nat.1 = k
                partial(cos_term(x + y), k.suc) =
                    partial(cauchy_seq(cos_term(x), cos_term(y)), k.suc) -
                    partial(cauchy_seq(sin_term(x), sin_term(y)), k.suc - Nat.1)
                p(k.suc)
            }
        }
    }
    p(Nat.0) and forall(k: Nat) { p(k) implies p(k.suc) }
    Nat.induction(p)
    p(n)
}

/// The sine of a sum is the sum of the cross products.
theorem sin_add(x: Real, y: Real) {
    (x + y).sin = x.sin * y.cos + x.cos * y.sin
} by {
    // The Cauchy products sum to the sine terms.
    forall(n: Nat) {
        sin_add_cauchy(x, y, n)
        cauchy_product(sin_term(x), cos_term(y), n) + cauchy_product(cos_term(x), sin_term(y), n) = sin_term(x + y, n)
        add_seq(cauchy_seq(sin_term(x), cos_term(y)), cauchy_seq(cos_term(x), sin_term(y)))(n) =
            cauchy_product(sin_term(x), cos_term(y), n) + cauchy_product(cos_term(x), sin_term(y), n)
        add_seq(cauchy_seq(sin_term(x), cos_term(y)), cauchy_seq(cos_term(x), sin_term(y)))(n) = sin_term(x + y, n)
    }
    add_seq(cauchy_seq(sin_term(x), cos_term(y)), cauchy_seq(cos_term(x), sin_term(y))) = sin_term(x + y)
    partial(add_seq(cauchy_seq(sin_term(x), cos_term(y)), cauchy_seq(cos_term(x), sin_term(y)))) = partial(sin_term(x + y))
    partial_add_seq_comm(cauchy_seq(sin_term(x), cos_term(y)), cauchy_seq(cos_term(x), sin_term(y)))
    partial(add_seq(cauchy_seq(sin_term(x), cos_term(y)), cauchy_seq(cos_term(x), sin_term(y)))) =
        add_seq(partial(cauchy_seq(sin_term(x), cos_term(y))), partial(cauchy_seq(cos_term(x), sin_term(y))))
    partial(sin_term(x + y)) =
        add_seq(partial(cauchy_seq(sin_term(x), cos_term(y))), partial(cauchy_seq(cos_term(x), sin_term(y))))

    // The Cauchy products converge to the products of the limits.
    sin_term_abs_converges(x)
    cos_term_abs_converges(y)
    cauchy_product_converges(sin_term(x), cos_term(y))
    converges_to(partial(cauchy_seq(sin_term(x), cos_term(y))), limit(partial(sin_term(x))) * limit(partial(cos_term(y))))
    x.sin = limit(partial(sin_term(x)))
    y.cos = limit(partial(cos_term(y)))
    converges_to(partial(cauchy_seq(sin_term(x), cos_term(y))), x.sin * y.cos)
    converges_to_imp_converges(partial(cauchy_seq(sin_term(x), cos_term(y))), x.sin * y.cos)
    converges(partial(cauchy_seq(sin_term(x), cos_term(y))))
    cos_term_abs_converges(x)
    sin_term_abs_converges(y)
    cauchy_product_converges(cos_term(x), sin_term(y))
    converges_to(partial(cauchy_seq(cos_term(x), sin_term(y))), limit(partial(cos_term(x))) * limit(partial(sin_term(y))))
    x.cos = limit(partial(cos_term(x)))
    y.sin = limit(partial(sin_term(y)))
    converges_to(partial(cauchy_seq(cos_term(x), sin_term(y))), x.cos * y.sin)
    converges_to_imp_converges(partial(cauchy_seq(cos_term(x), sin_term(y))), x.cos * y.sin)
    converges(partial(cauchy_seq(cos_term(x), sin_term(y))))

    // The sum of the partial sums converges to the sum of the limits.
    limit_add_seq(partial(cauchy_seq(sin_term(x), cos_term(y))), partial(cauchy_seq(cos_term(x), sin_term(y))))
    converges_to(add_seq(partial(cauchy_seq(sin_term(x), cos_term(y))), partial(cauchy_seq(cos_term(x), sin_term(y)))),
        limit(partial(cauchy_seq(sin_term(x), cos_term(y)))) + limit(partial(cauchy_seq(cos_term(x), sin_term(y)))))
    converges_to(partial(sin_term(x + y)), limit(partial(cauchy_seq(sin_term(x), cos_term(y)))) + limit(partial(cauchy_seq(cos_term(x), sin_term(y)))))
    sin_term_abs_converges(x + y)
    absolutely_converges(sin_term(x + y))
    absolutely_converges_imp_converges(sin_term(x + y))
    converges(partial(sin_term(x + y)))
    converges_imp_converges_to(partial(sin_term(x + y)))
    converges_to(partial(sin_term(x + y)), limit(partial(sin_term(x + y))))
    converges_to_unique(partial(sin_term(x + y)), limit(partial(cauchy_seq(sin_term(x), cos_term(y)))) + limit(partial(cauchy_seq(cos_term(x), sin_term(y)))), limit(partial(sin_term(x + y))))
    limit(partial(sin_term(x + y))) = limit(partial(cauchy_seq(sin_term(x), cos_term(y)))) + limit(partial(cauchy_seq(cos_term(x), sin_term(y))))

    // The limits of the Cauchy products are the products of the functions.
    converges_imp_converges_to(partial(cauchy_seq(sin_term(x), cos_term(y))))
    converges_to(partial(cauchy_seq(sin_term(x), cos_term(y))), limit(partial(cauchy_seq(sin_term(x), cos_term(y)))))
    converges_to_unique(partial(cauchy_seq(sin_term(x), cos_term(y))), x.sin * y.cos, limit(partial(cauchy_seq(sin_term(x), cos_term(y)))))
    limit(partial(cauchy_seq(sin_term(x), cos_term(y)))) = x.sin * y.cos
    converges_imp_converges_to(partial(cauchy_seq(cos_term(x), sin_term(y))))
    converges_to(partial(cauchy_seq(cos_term(x), sin_term(y))), limit(partial(cauchy_seq(cos_term(x), sin_term(y)))))
    converges_to_unique(partial(cauchy_seq(cos_term(x), sin_term(y))), x.cos * y.sin, limit(partial(cauchy_seq(cos_term(x), sin_term(y)))))
    limit(partial(cauchy_seq(cos_term(x), sin_term(y)))) = x.cos * y.sin

    limit(partial(sin_term(x + y))) = x.sin * y.cos + x.cos * y.sin
    (x + y).sin = limit(partial(sin_term(x + y)))
    (x + y).sin = x.sin * y.cos + x.cos * y.sin
}

/// The cosine of a sum is the difference of the products.
theorem cos_add(x: Real, y: Real) {
    (x + y).cos = x.cos * y.cos - x.sin * y.sin
} by {
    // The shifted sine-sine Cauchy partial sums converge to x.sin y.sin.
    define e(n: Nat) -> Real {
        partial(cauchy_seq(sin_term(x), sin_term(y)), n - Nat.1)
    }
    forall(i: Nat) {
        tail(e, Nat.1)(i) = e(Nat.1 + i)
        e(Nat.1 + i) = partial(cauchy_seq(sin_term(x), sin_term(y)), Nat.1 + i - Nat.1)
        Nat.1 + i - Nat.1 = i
        partial(cauchy_seq(sin_term(x), sin_term(y)), Nat.1 + i - Nat.1) = partial(cauchy_seq(sin_term(x), sin_term(y)), i)
        tail(e, Nat.1)(i) = partial(cauchy_seq(sin_term(x), sin_term(y)), i)
        partial(cauchy_seq(sin_term(x), sin_term(y)))(i) = partial(cauchy_seq(sin_term(x), sin_term(y)), i)
        tail(e, Nat.1)(i) = partial(cauchy_seq(sin_term(x), sin_term(y)))(i)
    }
    tail(e, Nat.1) = partial(cauchy_seq(sin_term(x), sin_term(y)))
    sin_term_abs_converges(x)
    sin_term_abs_converges(y)
    cauchy_product_converges(sin_term(x), sin_term(y))
    converges_to(partial(cauchy_seq(sin_term(x), sin_term(y))), limit(partial(sin_term(x))) * limit(partial(sin_term(y))))
    x.sin = limit(partial(sin_term(x)))
    y.sin = limit(partial(sin_term(y)))
    converges_to(partial(cauchy_seq(sin_term(x), sin_term(y))), x.sin * y.sin)
    converges_to_imp_converges(partial(cauchy_seq(sin_term(x), sin_term(y))), x.sin * y.sin)
    converges(partial(cauchy_seq(sin_term(x), sin_term(y))))
    converges(tail(e, Nat.1))
    tail_imp_converges_to(e, Nat.1)
    converges_to(e, limit(tail(e, Nat.1)))
    limit(tail(e, Nat.1)) = limit(partial(cauchy_seq(sin_term(x), sin_term(y))))
    converges_to(e, limit(partial(cauchy_seq(sin_term(x), sin_term(y)))))
    converges_imp_converges_to(partial(cauchy_seq(sin_term(x), sin_term(y))))
    converges_to(partial(cauchy_seq(sin_term(x), sin_term(y))), limit(partial(cauchy_seq(sin_term(x), sin_term(y)))))
    converges_to_unique(partial(cauchy_seq(sin_term(x), sin_term(y))), x.sin * y.sin, limit(partial(cauchy_seq(sin_term(x), sin_term(y)))))
    limit(partial(cauchy_seq(sin_term(x), sin_term(y)))) = x.sin * y.sin
    converges_to(e, x.sin * y.sin)
    converges_to_imp_converges(e, x.sin * y.sin)
    converges(e)

    // The cosine-cosine Cauchy partial sums converge to x.cos y.cos.
    cos_term_abs_converges(x)
    cos_term_abs_converges(y)
    cauchy_product_converges(cos_term(x), cos_term(y))
    converges_to(partial(cauchy_seq(cos_term(x), cos_term(y))), limit(partial(cos_term(x))) * limit(partial(cos_term(y))))
    x.cos = limit(partial(cos_term(x)))
    y.cos = limit(partial(cos_term(y)))
    converges_to(partial(cauchy_seq(cos_term(x), cos_term(y))), x.cos * y.cos)
    converges_to_imp_converges(partial(cauchy_seq(cos_term(x), cos_term(y))), x.cos * y.cos)
    converges(partial(cauchy_seq(cos_term(x), cos_term(y))))

    // The cosine partial sums of the sum equal the difference of the two.
    forall(n: Nat) {
        cos_add_cauchy_seq(x, y, n)
        partial(cos_term(x + y), n) =
            partial(cauchy_seq(cos_term(x), cos_term(y)), n) -
            partial(cauchy_seq(sin_term(x), sin_term(y)), n - Nat.1)
        e(n) = partial(cauchy_seq(sin_term(x), sin_term(y)), n - Nat.1)
        sub_seq(partial(cauchy_seq(cos_term(x), cos_term(y))), e, n) =
            partial(cauchy_seq(cos_term(x), cos_term(y)), n) - e(n)
        sub_seq(partial(cauchy_seq(cos_term(x), cos_term(y))), e, n) =
            partial(cauchy_seq(cos_term(x), cos_term(y)), n) - partial(cauchy_seq(sin_term(x), sin_term(y)), n - Nat.1)
        partial(cos_term(x + y), n) = sub_seq(partial(cauchy_seq(cos_term(x), cos_term(y))), e, n)
    }
    partial(cos_term(x + y)) = sub_seq(partial(cauchy_seq(cos_term(x), cos_term(y))), e)
    limit_sub_seq(partial(cauchy_seq(cos_term(x), cos_term(y))), e)
    limit(sub_seq(partial(cauchy_seq(cos_term(x), cos_term(y))), e)) =
        limit(partial(cauchy_seq(cos_term(x), cos_term(y)))) - limit(e)
    limit(partial(cos_term(x + y))) = limit(partial(cauchy_seq(cos_term(x), cos_term(y)))) - limit(e)
    converges_imp_converges_to(partial(cauchy_seq(cos_term(x), cos_term(y))))
    converges_to(partial(cauchy_seq(cos_term(x), cos_term(y))), limit(partial(cauchy_seq(cos_term(x), cos_term(y)))))
    converges_to_unique(partial(cauchy_seq(cos_term(x), cos_term(y))), x.cos * y.cos, limit(partial(cauchy_seq(cos_term(x), cos_term(y)))))
    limit(partial(cauchy_seq(cos_term(x), cos_term(y)))) = x.cos * y.cos
    converges_imp_converges_to(e)
    converges_to(e, limit(e))
    converges_to_unique(e, x.sin * y.sin, limit(e))
    limit(e) = x.sin * y.sin
    limit(partial(cos_term(x + y))) = x.cos * y.cos - x.sin * y.sin
    cos_term_abs_converges(x + y)
    absolutely_converges(cos_term(x + y))
    absolutely_converges_imp_converges(cos_term(x + y))
    converges(partial(cos_term(x + y)))
    (x + y).cos = limit(partial(cos_term(x + y)))
    (x + y).cos = x.cos * y.cos - x.sin * y.sin
}

/// The Pythagorean identity: the sum of the squares of sine and cosine is one.
theorem sin_sq_add_cos_sq(x: Real) {
    x.sin.pow(Nat.2) + x.cos.pow(Nat.2) = Real.1
} by {
    cos_add(x, -x)
    (x + (-x)).cos = x.cos * (-x).cos - x.sin * (-x).sin
    x + (-x) = Real.0
    (Real.0).cos = x.cos * (-x).cos - x.sin * (-x).sin
    cos_zero
    (Real.0).cos = Real.1
    Real.1 = x.cos * (-x).cos - x.sin * (-x).sin
    cos_neg(x)
    (-x).cos = x.cos
    sin_neg(x)
    (-x).sin = -x.sin
    Real.1 = x.cos * x.cos - x.sin * (-x.sin)
    x.sin * (-x.sin) = -(x.sin * x.sin)
    x.cos * x.cos - x.sin * (-x.sin) = x.cos * x.cos + x.sin * x.sin
    Real.1 = x.cos * x.cos + x.sin * x.sin
    pow_suc(x.sin, Nat.1)
    x.sin.pow(Nat.2) = x.sin * x.sin
    pow_suc(x.cos, Nat.1)
    x.cos.pow(Nat.2) = x.cos * x.cos
    Real.1 = x.cos.pow(Nat.2) + x.sin.pow(Nat.2)
    x.sin.pow(Nat.2) + x.cos.pow(Nat.2) = Real.1
}
