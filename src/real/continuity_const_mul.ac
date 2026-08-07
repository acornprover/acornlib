from data.basic.function_algebra import pointwise_mul
from real.continuity_composition import continuous_at_delta, continuous_at_intro,
    continuous_intro
from real.continuity_base import Real, continuous, continuous_at, continuous_condition
from real.derivative_linear import scalar_mul_close
from real.real_ring import exists_small_mul_variant

/// The function obtained by multiplying f on the left by the fixed real constant c.
define const_mul_left(c: Real, f: Real -> Real, x: Real) -> Real {
    c * f(x)
}

/// The function obtained by multiplying f on the right by the fixed real constant c.
define const_mul_right(f: Real -> Real, c: Real, x: Real) -> Real {
    f(x) * c
}

/// Multiplication by a fixed real constant on the left preserves continuity at a point.
theorem continuous_at_const_mul_left(c: Real, f: Real -> Real, x: Real) {
    continuous_at(f, x)
    implies continuous_at(const_mul_left(c, f), x)
} by {
    forall(eps: Real) {
        if eps.is_positive {
            exists_small_mul_variant(c, eps)
            let eps2: Real satisfy {
                eps2.is_positive and c.abs * eps2 < eps
            }
            continuous_at_delta(f, x, eps2)
            let delta: Real satisfy {
                delta.is_positive and continuous_condition(f, x, delta, eps2)
            }
            continuous_condition(f, x, delta, eps2) = forall(y: Real) {
                y.is_close(x, delta) implies f(y).is_close(f(x), eps2)
            }
            forall(y: Real) {
                if y.is_close(x, delta) {
                    f(y).is_close(f(x), eps2)
                    scalar_mul_close(c, f(y), f(x), eps2, eps)
                    (c * f(y)).is_close(c * f(x), eps)
                    const_mul_left(c, f, y) = c * f(y)
                    const_mul_left(c, f, x) = c * f(x)
                    const_mul_left(c, f, y).is_close(const_mul_left(c, f, x), eps)
                }
            }
            continuous_condition(const_mul_left(c, f), x, delta, eps)
            delta.is_positive and continuous_condition(const_mul_left(c, f), x, delta, eps)
            exists(d: Real) {
                d.is_positive and continuous_condition(const_mul_left(c, f), x, d, eps)
            }
        }
    }
    continuous_at_intro(const_mul_left(c, f), x)
    continuous_at(const_mul_left(c, f), x)
}

/// Multiplication by a fixed real constant on the left preserves continuous functions.
theorem continuous_const_mul_left(c: Real, f: Real -> Real) {
    continuous(f) implies continuous(const_mul_left(c, f))
} by {
    continuous(f) = forall(x: Real) {
        continuous_at(f, x)
    }
    forall(x: Real) {
        continuous_at(f, x)
        continuous_at_const_mul_left(c, f, x)
        continuous_at(const_mul_left(c, f), x)
    }
    continuous_intro(const_mul_left(c, f))
    continuous(const_mul_left(c, f))
}

/// Multiplication by a fixed real constant on the right preserves continuity at a point.
theorem continuous_at_const_mul_right(f: Real -> Real, c: Real, x: Real) {
    continuous_at(f, x)
    implies continuous_at(const_mul_right(f, c), x)
} by {
    forall(eps: Real) {
        if eps.is_positive {
            exists_small_mul_variant(c, eps)
            let eps2: Real satisfy {
                eps2.is_positive and c.abs * eps2 < eps
            }
            continuous_at_delta(f, x, eps2)
            let delta: Real satisfy {
                delta.is_positive and continuous_condition(f, x, delta, eps2)
            }
            continuous_condition(f, x, delta, eps2) = forall(y: Real) {
                y.is_close(x, delta) implies f(y).is_close(f(x), eps2)
            }
            forall(y: Real) {
                if y.is_close(x, delta) {
                    f(y).is_close(f(x), eps2)
                    scalar_mul_close(c, f(y), f(x), eps2, eps)
                    (c * f(y)).is_close(c * f(x), eps)
                    f(y) * c = c * f(y)
                    f(x) * c = c * f(x)
                    (f(y) * c).is_close(f(x) * c, eps)
                    const_mul_right(f, c, y) = f(y) * c
                    const_mul_right(f, c, x) = f(x) * c
                    const_mul_right(f, c, y).is_close(const_mul_right(f, c, x), eps)
                }
            }
            continuous_condition(const_mul_right(f, c), x, delta, eps)
            delta.is_positive and continuous_condition(const_mul_right(f, c), x, delta, eps)
            exists(d: Real) {
                d.is_positive and continuous_condition(const_mul_right(f, c), x, d, eps)
            }
        }
    }
    continuous_at_intro(const_mul_right(f, c), x)
    continuous_at(const_mul_right(f, c), x)
}

/// Multiplication by a fixed real constant on the right preserves continuous functions.
theorem continuous_const_mul_right(f: Real -> Real, c: Real) {
    continuous(f) implies continuous(const_mul_right(f, c))
} by {
    continuous(f) = forall(x: Real) {
        continuous_at(f, x)
    }
    forall(x: Real) {
        continuous_at(f, x)
        continuous_at_const_mul_right(f, c, x)
        continuous_at(const_mul_right(f, c), x)
    }
    continuous_intro(const_mul_right(f, c))
    continuous(const_mul_right(f, c))
}

/// The pointwise product with a constant function on the left agrees with const_mul_left.
theorem pointwise_mul_constant_left_eq(c: Real, f: Real -> Real) {
    pointwise_mul[Real, Real](constant[Real, Real](c), f) = const_mul_left(c, f)
} by {
    forall(x: Real) {
        constant[Real, Real](c, x) = c
        pointwise_mul[Real, Real](constant[Real, Real](c), f, x) = c * f(x)
        const_mul_left(c, f, x) = c * f(x)
        pointwise_mul[Real, Real](constant[Real, Real](c), f, x) = const_mul_left(c, f, x)
    }
}

/// The pointwise product with a constant function on the right agrees with const_mul_right.
theorem pointwise_mul_constant_right_eq(f: Real -> Real, c: Real) {
    pointwise_mul[Real, Real](f, constant[Real, Real](c)) = const_mul_right(f, c)
} by {
    forall(x: Real) {
        constant[Real, Real](c, x) = c
        pointwise_mul[Real, Real](f, constant[Real, Real](c), x) = f(x) * c
        const_mul_right(f, c, x) = f(x) * c
        pointwise_mul[Real, Real](f, constant[Real, Real](c), x) = const_mul_right(f, c, x)
    }
}
