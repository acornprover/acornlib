/// Shared infrastructure for continuous probability distributions.
///
/// Every continuous density in this package is built from the exponential,
/// so its integrability on a closed interval [a, b] follows from the same
/// pattern: the integrand has a continuous antiderivative (given by the
/// fundamental theorem of calculus), and the integrand itself is Lipschitz on
/// [a, b] because its derivative is bounded there (given by the mean value
/// theorem).  This file packages that pattern once so the distribution files
/// only have to supply the per-distribution antiderivative, derivative and
/// bound facts.
///
/// The lemmas here mirror real.lipschitz.derivative_bound_imp_lipschitz, but
/// with the derivative bound on the interval [a, b] instead of the whole
/// line: densities like t ↦ λ·e^(-λt) have derivatives that are bounded on
/// [0, ∞) but unbounded on (-∞, 0), so a global bound is unavailable.
///
///   - derivative_bound_imp_lipschitz_on_pair: one pair of points,
///   - derivative_bound_imp_lipschitz_on:      the pointwise Lipschitz bound,
///   - fn_integrable_gen_derivative_bound:      integrability of the integrand,
///   - integral_ftc2_derivative_bound:          integrability plus the value
///     g(b) - g(a) of the integral.

from order import lt_imp_lte, lte_trans, lte_antisymm, not_lt_imp_gte, not_lte_imp_gt, lt_imp_ne_symm
from real.continuity_base import Real, continuous
from real.derivative_basic import has_derivative_at, has_derivative_at_unique, sub_ne_zero_of_ne
from real.calculus_api import is_derivative_fn, is_derivative_fn_at, is_derivative_fn_iff, derivative_fn_identity, derivative_fn_const_mul, derivative_fn_square
from data.basic.functions import function_extensionality, function_eq_transport_predicate_rev, identity_fn
from data.basic.function_algebra import pointwise_mul, pointwise_add
from real.exp import one_half_plus_one_half
from real.continuity_const_mul import const_mul_left, continuous_const_mul_left
from real.continuity_pointwise_mul import continuous_pointwise_mul
from real.continuity_sequences import identity_function_is_continuous
from real.mean_value import mean_value_theorem, secant_slope
from real.real_base import abs_gte_zero, pos_gt_zero, gt_zero_imp_pos, pos_imp_eq_abs, abs_neg, lte_abs, add_comm, add_assoc
from real.real_seq import lt_imp_minus_pos
from real.real_ring import mul_zero_left, real_mul_comm, mul_nonneg, mul_le_mul_nonneg
from real.exp import abs_div
from ordered_field import multiply_inequality_with_nonnegative_element
from real.derivative_continuity import div_mul_cancel_denominator
from real.integral import integral, is_integrable, interval_contains, interval_contains_left, interval_contains_right
from real.integral_exp import ftc2_general
from real.integral_trig import fn_integrable_gen
from algebra.add_ordered_group import add_le_add_right

numerals Real

/// A function whose derivative is bounded by k on [a, b] is k-Lipschitz on
/// [a, b] at the pair (u, v).
theorem derivative_bound_imp_lipschitz_on_pair(
    f: Real -> Real, df: Real -> Real, k: Real, a: Real, b: Real, u: Real, v: Real
) {
    is_derivative_fn(f, df) and continuous(f) and Real.0 <= k and a <= b and
    (forall(z: Real) { interval_contains(a, b, z) implies df(z).abs <= k }) and
    interval_contains(a, b, u) and interval_contains(a, b, v)
    implies (f(u) - f(v)).abs <= k * (u - v).abs
} by {
    if is_derivative_fn(f, df) and continuous(f) and Real.0 <= k and a <= b and
       (forall(z: Real) { interval_contains(a, b, z) implies df(z).abs <= k }) and
       interval_contains(a, b, u) and interval_contains(a, b, v) {
        if u <= v {
            if u < v {
                mean_value_theorem(f, df, u, v)
                let c: Real satisfy {
                    u < c and c < v and has_derivative_at(f, c, secant_slope(f, u, v))
                }
                has_derivative_at(f, c, secant_slope(f, u, v))
                is_derivative_fn_at(f, df, c)
                has_derivative_at(f, c, df(c))
                has_derivative_at_unique(f, c, secant_slope(f, u, v), df(c))
                secant_slope(f, u, v) = df(c)
                secant_slope(f, u, v) = (f(v) - f(u)) / (v - u)
                df(c) = (f(v) - f(u)) / (v - u)
                interval_contains_left(a, b, u)
                a <= u
                lt_imp_lte(u, c)
                u <= c
                lte_trans(a, u, c)
                a <= c
                interval_contains_right(a, b, v)
                v <= b
                lt_imp_lte(c, v)
                c <= v
                lte_trans(c, v, b)
                c <= b
                a <= c and c <= b
                interval_contains(a, b, c)
                df(c).abs <= k
                abs_div(f(v) - f(u), v - u)
                ((f(v) - f(u)) / (v - u)).abs = (f(v) - f(u)).abs / (v - u).abs
                df(c).abs = (f(v) - f(u)).abs / (v - u).abs
                (f(v) - f(u)).abs / (v - u).abs <= k
                lt_imp_minus_pos(u, v)
                (v - u).is_positive
                pos_imp_eq_abs(v - u)
                v - u = (v - u).abs
                (v - u).abs = v - u
                (f(v) - f(u)).abs / (v - u) <= k
                pos_gt_zero(v - u)
                v - u > Real.0
                multiply_inequality_with_nonnegative_element[Real](
                    (f(v) - f(u)).abs / (v - u), k, v - u)
                ((f(v) - f(u)).abs / (v - u)) * (v - u) <= k * (v - u)
                lt_imp_ne_symm(u, v)
                v != u
                sub_ne_zero_of_ne(v, u)
                v - u != Real.0
                div_mul_cancel_denominator((f(v) - f(u)).abs, v - u)
                ((f(v) - f(u)).abs / (v - u)) * (v - u) = (f(v) - f(u)).abs
                (f(v) - f(u)).abs <= k * (v - u)
                (v - u).abs = v - u
                k * (v - u) = k * (v - u).abs
                (f(v) - f(u)).abs <= k * (v - u).abs
                f(v) - f(u) = -(f(u) - f(v))
                abs_neg(f(u) - f(v))
                (-(f(u) - f(v))).abs = (f(u) - f(v)).abs
                (f(v) - f(u)).abs = (f(u) - f(v)).abs
                v - u = -(u - v)
                abs_neg(u - v)
                (-(u - v)).abs = (u - v).abs
                (v - u).abs = (u - v).abs
                k * (v - u).abs = k * (u - v).abs
                (f(u) - f(v)).abs <= k * (u - v).abs
            } else {
                not_lt_imp_gte[Real](u, v)
                u >= v
                v <= u
                lte_antisymm[Real](u, v)
                u = v
                v = u
                f(u) = f(v)
                f(u) - f(v) = Real.0
                (f(u) - f(v)).abs = Real.0.abs
                Real.0.abs = Real.0
                (f(u) - f(v)).abs = Real.0
                abs_gte_zero(u - v)
                Real.0 <= (u - v).abs
                multiply_inequality_with_nonnegative_element[Real](Real.0, k, (u - v).abs)
                Real.0 * (u - v).abs <= k * (u - v).abs
                mul_zero_left((u - v).abs)
                Real.0 * (u - v).abs = Real.0
                Real.0 <= k * (u - v).abs
                (f(u) - f(v)).abs <= k * (u - v).abs
            }
        } else {
            not_lte_imp_gt[Real](u, v)
            u > v
            v < u
            lt_imp_lte(v, u)
            v <= u
            mean_value_theorem(f, df, v, u)
            let c: Real satisfy {
                v < c and c < u and has_derivative_at(f, c, secant_slope(f, v, u))
            }
                has_derivative_at(f, c, secant_slope(f, v, u))
                is_derivative_fn_at(f, df, c)
                has_derivative_at(f, c, df(c))
                has_derivative_at_unique(f, c, secant_slope(f, v, u), df(c))
                secant_slope(f, v, u) = df(c)
                secant_slope(f, v, u) = (f(u) - f(v)) / (u - v)
                df(c) = (f(u) - f(v)) / (u - v)
                interval_contains_left(a, b, v)
                a <= v
                lt_imp_lte(v, c)
                v <= c
                lte_trans(a, v, c)
                a <= c
                interval_contains_right(a, b, u)
                u <= b
                lt_imp_lte(c, u)
                c <= u
                lte_trans(c, u, b)
                c <= b
                a <= c and c <= b
                interval_contains(a, b, c)
                df(c).abs <= k
                abs_div(f(u) - f(v), u - v)
                ((f(u) - f(v)) / (u - v)).abs = (f(u) - f(v)).abs / (u - v).abs
                df(c).abs = (f(u) - f(v)).abs / (u - v).abs
                (f(u) - f(v)).abs / (u - v).abs <= k
                lt_imp_minus_pos(v, u)
                (u - v).is_positive
                pos_imp_eq_abs(u - v)
                u - v = (u - v).abs
                (u - v).abs = u - v
                (f(u) - f(v)).abs / (u - v) <= k
                pos_gt_zero(u - v)
                u - v > Real.0
                multiply_inequality_with_nonnegative_element[Real](
                    (f(u) - f(v)).abs / (u - v), k, u - v)
                ((f(u) - f(v)).abs / (u - v)) * (u - v) <= k * (u - v)
                lt_imp_ne_symm(v, u)
                u != v
                sub_ne_zero_of_ne(u, v)
                u - v != Real.0
                div_mul_cancel_denominator((f(u) - f(v)).abs, u - v)
                ((f(u) - f(v)).abs / (u - v)) * (u - v) = (f(u) - f(v)).abs
                (f(u) - f(v)).abs <= k * (u - v)
                (u - v).abs = u - v
                k * (u - v) = k * (u - v).abs
                (f(u) - f(v)).abs <= k * (u - v).abs
        }
    }
}

/// A function whose derivative is bounded by k on [a, b] is k-Lipschitz on
/// [a, b].
theorem derivative_bound_imp_lipschitz_on(
    f: Real -> Real, df: Real -> Real, k: Real, a: Real, b: Real
) {
    is_derivative_fn(f, df) and continuous(f) and Real.0 <= k and a <= b and
    (forall(z: Real) { interval_contains(a, b, z) implies df(z).abs <= k })
    implies forall(u: Real, v: Real) {
        interval_contains(a, b, u) and interval_contains(a, b, v) implies
        (f(u) - f(v)).abs <= k * (u - v).abs
    }
} by {
    if is_derivative_fn(f, df) and continuous(f) and Real.0 <= k and a <= b and
       (forall(z: Real) { interval_contains(a, b, z) implies df(z).abs <= k }) {
        forall(u: Real, v: Real) {
            if interval_contains(a, b, u) and interval_contains(a, b, v) {
                derivative_bound_imp_lipschitz_on_pair(f, df, k, a, b, u, v)
                (f(u) - f(v)).abs <= k * (u - v).abs
            }
        }
    }
}

/// An integrand with a continuous antiderivative, a derivative bounded on
/// [a, b], and bounds on [a, b] is integrable on [a, b].
theorem fn_integrable_gen_derivative_bound(
    f: Real -> Real, g: Real -> Real, df: Real -> Real, a: Real, b: Real,
    k: Real, lb: Real, ub: Real
) {
    a <= b and Real.0 <= k and continuous(g) and is_derivative_fn(g, f) and
    is_derivative_fn(f, df) and continuous(f) and
    (forall(z: Real) { interval_contains(a, b, z) implies df(z).abs <= k }) and
    (forall(t: Real) { interval_contains(a, b, t) implies lb <= f(t) }) and
    (forall(t: Real) { interval_contains(a, b, t) implies f(t) <= ub })
    implies is_integrable(f, a, b)
} by {
    if a <= b and Real.0 <= k and continuous(g) and is_derivative_fn(g, f) and
       is_derivative_fn(f, df) and continuous(f) and
       (forall(z: Real) { interval_contains(a, b, z) implies df(z).abs <= k }) and
       (forall(t: Real) { interval_contains(a, b, t) implies lb <= f(t) }) and
       (forall(t: Real) { interval_contains(a, b, t) implies f(t) <= ub }) {
        derivative_bound_imp_lipschitz_on(f, df, k, a, b)
        forall(u: Real, v: Real) {
            if interval_contains(a, b, u) and interval_contains(a, b, v) {
                (f(u) - f(v)).abs <= k * (u - v).abs
            }
        }
        fn_integrable_gen(f, g, a, b, k, lb, ub)
        is_integrable(f, a, b)
    }
}

/// An integrand with a continuous antiderivative, a derivative bounded on
/// [a, b], and bounds on [a, b] is integrable on [a, b] with integral
/// g(b) - g(a).
theorem integral_ftc2_derivative_bound(
    f: Real -> Real, g: Real -> Real, df: Real -> Real, a: Real, b: Real,
    k: Real, lb: Real, ub: Real
) {
    a <= b and Real.0 <= k and continuous(g) and is_derivative_fn(g, f) and
    is_derivative_fn(f, df) and continuous(f) and
    (forall(z: Real) { interval_contains(a, b, z) implies df(z).abs <= k }) and
    (forall(t: Real) { interval_contains(a, b, t) implies lb <= f(t) }) and
    (forall(t: Real) { interval_contains(a, b, t) implies f(t) <= ub })
    implies (is_integrable(f, a, b) and integral(f, a, b) = g(b) - g(a))
} by {
    if a <= b and Real.0 <= k and continuous(g) and is_derivative_fn(g, f) and
       is_derivative_fn(f, df) and continuous(f) and
       (forall(z: Real) { interval_contains(a, b, z) implies df(z).abs <= k }) and
       (forall(t: Real) { interval_contains(a, b, t) implies lb <= f(t) }) and
       (forall(t: Real) { interval_contains(a, b, t) implies f(t) <= ub }) {
        fn_integrable_gen_derivative_bound(f, g, df, a, b, k, lb, ub)
        is_integrable(f, a, b)
        ftc2_general(f, g, a, b, lb, ub)
        integral(f, a, b) = g(b) - g(a)
        is_integrable(f, a, b) and integral(f, a, b) = g(b) - g(a)
    }
}

/// If a <= b then the difference b - a is nonnegative.
theorem sub_nonneg_of_lte(a: Real, b: Real) {
    a <= b implies Real.0 <= b - a
} by {
    if a <= b {
        add_le_add_right(a, b, -a)
        a + -a <= b + -a
        a + -a = Real.0
        b + -a = b - a
        Real.0 <= b - a
    }
}

/// If a is nonnegative then b - a is at most b.
theorem lte_sub_of_nonneg(a: Real, b: Real) {
    Real.0 <= a implies b - a <= b
} by {
    if Real.0 <= a {
        add_le_add_right(Real.0, a, -a)
        Real.0 + -a <= a + -a
        Real.0 + -a = -a
        a + -a = Real.0
        -a <= Real.0
        add_le_add_right(-a, Real.0, b)
        -a + b <= Real.0 + b
        -a + b = b - a
        Real.0 + b = b
        b - a <= b
    }
}

/// If the absolute value of a is at most c then a is at most c.
theorem lte_of_abs_le(a: Real, c: Real) {
    a.abs <= c implies a <= c
} by {
    if a.abs <= c {
        lte_abs(a)
        a <= a.abs
        lte_trans(a, a.abs, c)
        a <= c
    }
}

/// If the absolute value of a is at most c then -c is at most a.
theorem neg_lte_of_abs_le(a: Real, c: Real) {
    a.abs <= c implies -c <= a
} by {
    if a.abs <= c {
        lte_abs(-a)
        -a <= (-a).abs
        abs_neg(a)
        (-a).abs = a.abs
        -a <= a.abs
        lte_trans(-a, a.abs, c)
        -a <= c
        add_le_add_right(-a, c, a)
        -a + a <= c + a
        -a + a = Real.0
        Real.0 <= c + a
        add_le_add_right(Real.0, c + a, -c)
        Real.0 + -c <= c + a + -c
        Real.0 + -c = -c
        add_assoc(c, a, -c)
        c + a + -c = c + (a + -c)
        add_comm(a, -c)
        a + -c = -c + a
        c + (a + -c) = c + (-c + a)
        add_assoc(c, -c, a)
        (c + -c) + a = c + (-c + a)
        c + -c = Real.0
        (c + -c) + a = Real.0 + a
        Real.0 + a = a
        c + a + -c = a
        -c <= a
    }
}

/// A global derivative transports along pointwise equality of both the
/// function and the derivative.
theorem is_derivative_fn_both_eq(f: Real -> Real, g: Real -> Real, df: Real -> Real, dg: Real -> Real) {
    is_derivative_fn(f, df) and (forall(x: Real) { f(x) = g(x) }) and (forall(x: Real) { df(x) = dg(x) })
    implies is_derivative_fn(g, dg)
} by {
    if is_derivative_fn(f, df) and (forall(x: Real) { f(x) = g(x) }) and (forall(x: Real) { df(x) = dg(x) }) {
        forall(x: Real) {
            is_derivative_fn_at(f, df, x)
            has_derivative_at(f, x, df(x))
            df(x) = dg(x)
            has_derivative_at(f, x, dg(x))
        }
        is_derivative_fn_iff(f, dg)
        is_derivative_fn(f, dg) = forall(y: Real) {
            has_derivative_at(f, y, dg(y))
        }
        is_derivative_fn(f, dg)
        function_extensionality(f, g)
        f = g
        define derivative_pred(h: Real -> Real) -> Bool {
            is_derivative_fn(h, dg)
        }
        derivative_pred(f)
        function_eq_transport_predicate_rev(derivative_pred, g, f)
        is_derivative_fn(g, dg)
    }
}

/// The pointwise square x ↦ x·x has derivative x ↦ x + x.
theorem sq_id_derivative {
    is_derivative_fn(pointwise_mul(identity_fn[Real], identity_fn[Real]),
        pointwise_add(pointwise_mul(identity_fn[Real], constant[Real, Real](Real.1)),
            pointwise_mul(identity_fn[Real], constant[Real, Real](Real.1))))
} by {
    derivative_fn_identity
    is_derivative_fn(identity_fn[Real], constant[Real, Real](Real.1))
    derivative_fn_square(identity_fn[Real], constant[Real, Real](Real.1))
    is_derivative_fn(pointwise_mul(identity_fn[Real], identity_fn[Real]),
        pointwise_add(pointwise_mul(identity_fn[Real], constant[Real, Real](Real.1)),
            pointwise_mul(identity_fn[Real], constant[Real, Real](Real.1))))
}

/// The half square x ↦ x²/2 has derivative x ↦ x.
theorem half_sq_id_derivative {
    is_derivative_fn(const_mul_left(Real.one_half,
        pointwise_mul(identity_fn[Real], identity_fn[Real])), identity_fn[Real])
} by {
    sq_id_derivative
    is_derivative_fn(pointwise_mul(identity_fn[Real], identity_fn[Real]),
        pointwise_add(pointwise_mul(identity_fn[Real], constant[Real, Real](Real.1)),
            pointwise_mul(identity_fn[Real], constant[Real, Real](Real.1))))
    derivative_fn_const_mul(Real.one_half,
        pointwise_mul(identity_fn[Real], identity_fn[Real]),
        pointwise_add(pointwise_mul(identity_fn[Real], constant[Real, Real](Real.1)),
            pointwise_mul(identity_fn[Real], constant[Real, Real](Real.1))))
    is_derivative_fn(pointwise_mul(constant[Real, Real](Real.one_half),
        pointwise_mul(identity_fn[Real], identity_fn[Real])),
        pointwise_mul(constant[Real, Real](Real.one_half),
            pointwise_add(pointwise_mul(identity_fn[Real], constant[Real, Real](Real.1)),
                pointwise_mul(identity_fn[Real], constant[Real, Real](Real.1)))))
    forall(x: Real) {
        pointwise_mul(constant[Real, Real](Real.one_half),
            pointwise_mul(identity_fn[Real], identity_fn[Real]), x) =
            constant[Real, Real](Real.one_half, x) *
                pointwise_mul(identity_fn[Real], identity_fn[Real], x)
        constant[Real, Real](Real.one_half, x) = Real.one_half
        const_mul_left(Real.one_half,
            pointwise_mul(identity_fn[Real], identity_fn[Real]), x) =
            Real.one_half * pointwise_mul(identity_fn[Real], identity_fn[Real], x)
        pointwise_mul(constant[Real, Real](Real.one_half),
            pointwise_mul(identity_fn[Real], identity_fn[Real]), x) =
            const_mul_left(Real.one_half,
                pointwise_mul(identity_fn[Real], identity_fn[Real]), x)
    }
    function_extensionality(pointwise_mul(constant[Real, Real](Real.one_half),
        pointwise_mul(identity_fn[Real], identity_fn[Real])),
        const_mul_left(Real.one_half,
            pointwise_mul(identity_fn[Real], identity_fn[Real])))
    forall(x: Real) {
        pointwise_mul(constant[Real, Real](Real.one_half),
            pointwise_add(pointwise_mul(identity_fn[Real], constant[Real, Real](Real.1)),
                pointwise_mul(identity_fn[Real], constant[Real, Real](Real.1))), x) =
            constant[Real, Real](Real.one_half, x) *
                pointwise_add(pointwise_mul(identity_fn[Real], constant[Real, Real](Real.1)),
                    pointwise_mul(identity_fn[Real], constant[Real, Real](Real.1)), x)
        constant[Real, Real](Real.one_half, x) = Real.one_half
        pointwise_add(pointwise_mul(identity_fn[Real], constant[Real, Real](Real.1)),
            pointwise_mul(identity_fn[Real], constant[Real, Real](Real.1)), x) =
            pointwise_mul(identity_fn[Real], constant[Real, Real](Real.1), x) +
            pointwise_mul(identity_fn[Real], constant[Real, Real](Real.1), x)
        pointwise_mul(identity_fn[Real], constant[Real, Real](Real.1), x) =
            identity_fn[Real](x) * constant[Real, Real](Real.1, x)
        identity_fn[Real](x) = x
        constant[Real, Real](Real.1, x) = Real.1
        pointwise_mul(identity_fn[Real], constant[Real, Real](Real.1), x) = x
        pointwise_add(pointwise_mul(identity_fn[Real], constant[Real, Real](Real.1)),
            pointwise_mul(identity_fn[Real], constant[Real, Real](Real.1)), x) = x + x
        Real.one_half * (x + x) = Real.one_half * x + Real.one_half * x
        Real.one_half * x + Real.one_half * x = (Real.one_half + Real.one_half) * x
        one_half_plus_one_half
        Real.one_half + Real.one_half = Real.1
        (Real.one_half + Real.one_half) * x = Real.1 * x
        Real.1 * x = x
        Real.one_half * (x + x) = x
        pointwise_mul(constant[Real, Real](Real.one_half),
            pointwise_add(pointwise_mul(identity_fn[Real], constant[Real, Real](Real.1)),
                pointwise_mul(identity_fn[Real], constant[Real, Real](Real.1))), x) = x
        identity_fn[Real](x) = x
        pointwise_mul(constant[Real, Real](Real.one_half),
            pointwise_add(pointwise_mul(identity_fn[Real], constant[Real, Real](Real.1)),
                pointwise_mul(identity_fn[Real], constant[Real, Real](Real.1))), x) =
            identity_fn[Real](x)
    }
    function_extensionality(pointwise_mul(constant[Real, Real](Real.one_half),
        pointwise_add(pointwise_mul(identity_fn[Real], constant[Real, Real](Real.1)),
            pointwise_mul(identity_fn[Real], constant[Real, Real](Real.1)))),
        identity_fn[Real])
    is_derivative_fn_both_eq(pointwise_mul(constant[Real, Real](Real.one_half),
        pointwise_mul(identity_fn[Real], identity_fn[Real])),
        const_mul_left(Real.one_half,
            pointwise_mul(identity_fn[Real], identity_fn[Real])),
        pointwise_mul(constant[Real, Real](Real.one_half),
            pointwise_add(pointwise_mul(identity_fn[Real], constant[Real, Real](Real.1)),
                pointwise_mul(identity_fn[Real], constant[Real, Real](Real.1)))),
        identity_fn[Real])
    is_derivative_fn(const_mul_left(Real.one_half,
        pointwise_mul(identity_fn[Real], identity_fn[Real])), identity_fn[Real])
}

/// The half square x ↦ x²/2 is continuous.
theorem half_sq_id_continuous {
    continuous(const_mul_left(Real.one_half,
        pointwise_mul(identity_fn[Real], identity_fn[Real])))
} by {
    identity_function_is_continuous
    continuous(identity_fn[Real])
    continuous_pointwise_mul(identity_fn[Real], identity_fn[Real])
    continuous(pointwise_mul(identity_fn[Real], identity_fn[Real]))
    continuous_const_mul_left(Real.one_half,
        pointwise_mul(identity_fn[Real], identity_fn[Real]))
    continuous(const_mul_left(Real.one_half,
        pointwise_mul(identity_fn[Real], identity_fn[Real])))
}

/// If both w and -w are at most c then the absolute value of w is at most c.
theorem abs_le_of_lte_and_neg_lte(w: Real, c: Real) {
    w <= c and -w <= c implies w.abs <= c
} by {
    if w.is_negative {
        w.abs = -w
        -w <= c
        w.abs <= c
    }
    if not w.is_negative {
        w.abs = w
        w <= c
        w.abs <= c
    }
    w.is_negative or not w.is_negative
    w.abs <= c
}

/// The product of two nonnegative numbers is nonnegative.
theorem mul_nonneg_lte(a: Real, b: Real) {
    Real.0 <= a and Real.0 <= b implies Real.0 <= a * b
} by {
    if Real.0 <= a and Real.0 <= b {
        a >= Real.0
        b >= Real.0
        mul_nonneg(a, b)
        a * b >= Real.0
        Real.0 <= a * b
    }
}

/// Pointwise products of nonnegative, ordered pairs.
theorem mul_le_mul_nonneg_lte(a: Real, b: Real, c: Real, d: Real) {
    Real.0 <= a and Real.0 <= b and Real.0 <= c and Real.0 <= d and a <= c and b <= d
    implies a * b <= c * d
} by {
    if Real.0 <= a and Real.0 <= b and Real.0 <= c and Real.0 <= d and a <= c and b <= d {
        a >= Real.0
        b >= Real.0
        c >= Real.0
        d >= Real.0
        mul_le_mul_nonneg(a, b, c, d)
        a * b <= c * d
    }
}

/// The pointwise sum of two derivative functions commutes.
theorem is_derivative_fn_pointwise_add_comm(f: Real -> Real, a: Real -> Real, b: Real -> Real) {
    is_derivative_fn(f, pointwise_add(a, b))
    implies is_derivative_fn(f, pointwise_add(b, a))
} by {
    if is_derivative_fn(f, pointwise_add(a, b)) {
        forall(x: Real) {
            pointwise_add(a, b, x) = a(x) + b(x)
            a(x) + b(x) = b(x) + a(x)
            pointwise_add(b, a, x) = b(x) + a(x)
            pointwise_add(a, b, x) = pointwise_add(b, a, x)
        }
        function_extensionality(pointwise_add(a, b), pointwise_add(b, a))
        is_derivative_fn_both_eq(f, f, pointwise_add(a, b), pointwise_add(b, a))
        is_derivative_fn(f, pointwise_add(b, a))
    }
}
