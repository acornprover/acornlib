/// Shallow linear-combination consumers for real asymptotic boundedness and vanishing.
///
/// This module intentionally does not introduce public big-O/little-o notation,
/// another eventual predicate API, or interface-barrel exports. It packages
/// common algebraic combinations using the published `real.asymptotic_bounds`
/// endpoints.

from nat import Nat
from real.real_base import Real
from real.real_seq import add_seq
from real.real_series import mul_seq, neg_seq
from real.prod_seq import prod_seq
from real.abs_conv import sub_seq
from real.double_sum import sub_seq_eq_add_neg
from real.bounded_seq import is_bounded_seq
from real.limits import vanishes
from real.asymptotic_bounds import bounded_neg_seq, bounded_add_seq, bounded_mul_seq,
    bounded_prod_seq, bounded_mul_vanishing_seq, vanishing_mul_bounded_seq,
    vanishes_neg_seq, vanishes_add_seq, vanishes_mul_seq

/// The pointwise difference of two bounded sequences is bounded.
theorem bounded_sub_seq(a: Nat -> Real, b: Nat -> Real) {
    is_bounded_seq(a) and is_bounded_seq(b) implies is_bounded_seq(sub_seq(a, b))
} by {
    if is_bounded_seq(a) and is_bounded_seq(b) {
        bounded_neg_seq(b)
        is_bounded_seq(neg_seq(b))
        bounded_add_seq(a, neg_seq(b))
        is_bounded_seq(add_seq(a, neg_seq(b)))
        sub_seq_eq_add_neg(a, b)
        sub_seq(a, b) = add_seq(a, neg_seq(b))
        is_bounded_seq(sub_seq(a, b))
    }
}

/// The pointwise difference of two vanishing sequences vanishes.
theorem vanishes_sub_seq(a: Nat -> Real, b: Nat -> Real) {
    vanishes(a) and vanishes(b) implies vanishes(sub_seq(a, b))
} by {
    if vanishes(a) and vanishes(b) {
        vanishes_neg_seq(b)
        vanishes(neg_seq(b))
        vanishes_add_seq(a, neg_seq(b))
        vanishes(add_seq(a, neg_seq(b)))
        sub_seq_eq_add_neg(a, b)
        sub_seq(a, b) = add_seq(a, neg_seq(b))
        vanishes(sub_seq(a, b))
    }
}

/// A two-term scalar linear combination of bounded sequences is bounded.
theorem bounded_linear_combination_two(a: Nat -> Real, b: Nat -> Real, c: Real, d: Real) {
    is_bounded_seq(a) and is_bounded_seq(b) implies is_bounded_seq(add_seq(mul_seq(c, a), mul_seq(d, b)))
} by {
    if is_bounded_seq(a) and is_bounded_seq(b) {
        bounded_mul_seq(c, a)
        is_bounded_seq(mul_seq(c, a))
        bounded_mul_seq(d, b)
        is_bounded_seq(mul_seq(d, b))
        bounded_add_seq(mul_seq(c, a), mul_seq(d, b))
        is_bounded_seq(add_seq(mul_seq(c, a), mul_seq(d, b)))
    }
}

/// A two-term scalar linear combination of vanishing sequences vanishes.
theorem vanishes_linear_combination_two(a: Nat -> Real, b: Nat -> Real, c: Real, d: Real) {
    vanishes(a) and vanishes(b) implies vanishes(add_seq(mul_seq(c, a), mul_seq(d, b)))
} by {
    if vanishes(a) and vanishes(b) {
        vanishes_mul_seq(c, a)
        vanishes(mul_seq(c, a))
        vanishes_mul_seq(d, b)
        vanishes(mul_seq(d, b))
        vanishes_add_seq(mul_seq(c, a), mul_seq(d, b))
        vanishes(add_seq(mul_seq(c, a), mul_seq(d, b)))
    }
}

/// A scalar multiple of a bounded sequence plus a bounded sequence is bounded.
theorem bounded_scalar_add_seq(a: Nat -> Real, b: Nat -> Real, c: Real) {
    is_bounded_seq(a) and is_bounded_seq(b) implies is_bounded_seq(add_seq(mul_seq(c, a), b))
} by {
    if is_bounded_seq(a) and is_bounded_seq(b) {
        bounded_mul_seq(c, a)
        is_bounded_seq(mul_seq(c, a))
        bounded_add_seq(mul_seq(c, a), b)
        is_bounded_seq(add_seq(mul_seq(c, a), b))
    }
}

/// A scalar multiple of a vanishing sequence plus a vanishing sequence vanishes.
theorem vanishes_scalar_add_seq(a: Nat -> Real, b: Nat -> Real, c: Real) {
    vanishes(a) and vanishes(b) implies vanishes(add_seq(mul_seq(c, a), b))
} by {
    if vanishes(a) and vanishes(b) {
        vanishes_mul_seq(c, a)
        vanishes(mul_seq(c, a))
        vanishes_add_seq(mul_seq(c, a), b)
        vanishes(add_seq(mul_seq(c, a), b))
    }
}

/// The product of two bounded two-term linear combinations is bounded.
theorem bounded_prod_linear_combinations_two(
    a: Nat -> Real,
    b: Nat -> Real,
    c: Nat -> Real,
    d: Nat -> Real,
    alpha: Real,
    beta: Real,
    gamma: Real,
    delta: Real
) {
    is_bounded_seq(a) and is_bounded_seq(b) and is_bounded_seq(c) and is_bounded_seq(d)
    implies is_bounded_seq(prod_seq(add_seq(mul_seq(alpha, a), mul_seq(beta, b)),
        add_seq(mul_seq(gamma, c), mul_seq(delta, d))))
} by {
    if is_bounded_seq(a) and is_bounded_seq(b) and is_bounded_seq(c) and is_bounded_seq(d) {
        bounded_linear_combination_two(a, b, alpha, beta)
        is_bounded_seq(add_seq(mul_seq(alpha, a), mul_seq(beta, b)))
        bounded_linear_combination_two(c, d, gamma, delta)
        is_bounded_seq(add_seq(mul_seq(gamma, c), mul_seq(delta, d)))
        bounded_prod_seq(add_seq(mul_seq(alpha, a), mul_seq(beta, b)),
            add_seq(mul_seq(gamma, c), mul_seq(delta, d)))
        is_bounded_seq(prod_seq(add_seq(mul_seq(alpha, a), mul_seq(beta, b)),
            add_seq(mul_seq(gamma, c), mul_seq(delta, d))))
    }
}

/// A bounded two-term linear combination times a vanishing sequence vanishes.
theorem bounded_linear_combination_mul_vanishing(
    a: Nat -> Real,
    b: Nat -> Real,
    c: Nat -> Real,
    alpha: Real,
    beta: Real
) {
    is_bounded_seq(a) and is_bounded_seq(b) and vanishes(c)
    implies vanishes(prod_seq(add_seq(mul_seq(alpha, a), mul_seq(beta, b)), c))
} by {
    if is_bounded_seq(a) and is_bounded_seq(b) and vanishes(c) {
        bounded_linear_combination_two(a, b, alpha, beta)
        is_bounded_seq(add_seq(mul_seq(alpha, a), mul_seq(beta, b)))
        bounded_mul_vanishing_seq(add_seq(mul_seq(alpha, a), mul_seq(beta, b)), c)
        vanishes(prod_seq(add_seq(mul_seq(alpha, a), mul_seq(beta, b)), c))
    }
}

/// A vanishing sequence times a bounded two-term linear combination vanishes.
theorem vanishing_mul_bounded_linear_combination(
    a: Nat -> Real,
    b: Nat -> Real,
    c: Nat -> Real,
    alpha: Real,
    beta: Real
) {
    vanishes(a) and is_bounded_seq(b) and is_bounded_seq(c)
    implies vanishes(prod_seq(a, add_seq(mul_seq(alpha, b), mul_seq(beta, c))))
} by {
    if vanishes(a) and is_bounded_seq(b) and is_bounded_seq(c) {
        bounded_linear_combination_two(b, c, alpha, beta)
        is_bounded_seq(add_seq(mul_seq(alpha, b), mul_seq(beta, c)))
        vanishing_mul_bounded_seq(a, add_seq(mul_seq(alpha, b), mul_seq(beta, c)))
        vanishes(prod_seq(a, add_seq(mul_seq(alpha, b), mul_seq(beta, c))))
    }
}

/// The product of a vanishing two-term linear combination and a bounded two-term linear combination vanishes.
theorem vanishing_linear_combination_mul_bounded_linear_combination(
    a: Nat -> Real,
    b: Nat -> Real,
    c: Nat -> Real,
    d: Nat -> Real,
    alpha: Real,
    beta: Real,
    gamma: Real,
    delta: Real
) {
    vanishes(a) and vanishes(b) and is_bounded_seq(c) and is_bounded_seq(d)
    implies vanishes(prod_seq(add_seq(mul_seq(alpha, a), mul_seq(beta, b)),
        add_seq(mul_seq(gamma, c), mul_seq(delta, d))))
} by {
    if vanishes(a) and vanishes(b) and is_bounded_seq(c) and is_bounded_seq(d) {
        vanishes_linear_combination_two(a, b, alpha, beta)
        vanishes(add_seq(mul_seq(alpha, a), mul_seq(beta, b)))
        bounded_linear_combination_two(c, d, gamma, delta)
        is_bounded_seq(add_seq(mul_seq(gamma, c), mul_seq(delta, d)))
        vanishing_mul_bounded_seq(add_seq(mul_seq(alpha, a), mul_seq(beta, b)),
            add_seq(mul_seq(gamma, c), mul_seq(delta, d)))
        vanishes(prod_seq(add_seq(mul_seq(alpha, a), mul_seq(beta, b)),
            add_seq(mul_seq(gamma, c), mul_seq(delta, d))))
    }
}
