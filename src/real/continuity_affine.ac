from data.basic.functions import identity_fn
from real.continuity_composition import continuous_imp_continuous_at
from real.continuity_const_add import const_add_right,
    continuous_at_const_add_right, continuous_const_add_right
from real.continuity_const_mul import const_mul_left,
    continuous_at_const_mul_left, continuous_const_mul_left
from real.continuity_sequences import identity_function_is_continuous
from real.continuity_base import Real, continuous, continuous_at

/// The affine real function x -> a * x + b.
define affine_real(a: Real, b: Real, x: Real) -> Real {
    a * x + b
}

/// The affine function agrees with adding b on the right of multiplying the identity by a.
theorem affine_real_eq_const_add_const_mul(a: Real, b: Real) {
    affine_real(a, b) = const_add_right(const_mul_left(a, identity_fn[Real]), b)
} by {
    forall(x: Real) {
        identity_fn[Real](x) = x
        const_mul_left(a, identity_fn[Real], x) = a * x
        const_add_right(const_mul_left(a, identity_fn[Real]), b, x) = a * x + b
        affine_real(a, b, x) = a * x + b
        affine_real(a, b, x) = const_add_right(const_mul_left(a, identity_fn[Real]), b, x)
    }
}

/// The affine function x -> a * x + b is continuous at each point.
theorem continuous_at_affine_real(a: Real, b: Real, x: Real) {
    continuous_at(affine_real(a, b), x)
} by {
    identity_function_is_continuous
    continuous_imp_continuous_at(identity_fn[Real], x)
    continuous_at(identity_fn[Real], x)
    continuous_at_const_mul_left(a, identity_fn[Real], x)
    continuous_at(const_mul_left(a, identity_fn[Real]), x)
    continuous_at_const_add_right(const_mul_left(a, identity_fn[Real]), b, x)
    continuous_at(const_add_right(const_mul_left(a, identity_fn[Real]), b), x)
    affine_real_eq_const_add_const_mul(a, b)
    continuous_at(affine_real(a, b), x)
}

/// The affine function x -> a * x + b is continuous.
theorem continuous_affine_real(a: Real, b: Real) {
    continuous(affine_real(a, b))
} by {
    identity_function_is_continuous
    continuous(identity_fn[Real])
    continuous_const_mul_left(a, identity_fn[Real])
    continuous(const_mul_left(a, identity_fn[Real]))
    continuous_const_add_right(const_mul_left(a, identity_fn[Real]), b)
    continuous(const_add_right(const_mul_left(a, identity_fn[Real]), b))
    affine_real_eq_const_add_const_mul(a, b)
    continuous(affine_real(a, b))
}
