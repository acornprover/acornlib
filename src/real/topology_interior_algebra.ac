from data.basic.set import Set, double_inclusion, intersection_contains_intro, set_supset_contains_intersection,
    sets_subset_contain_union, sets_subset_intersection, sets_subset_union
from real.real_field import Real
from real.topology import interior, interior_contains_eq, interior_point_intro,
    interior_subset, is_interior_point, is_open_set, open_set_interior_point
from real.topology_open_intersection import interior_point_intersection

/// Interior is monotone with respect to subset inclusion.
theorem interior_mono(s: Set[Real], t: Set[Real]) {
    s.subset(t) implies interior(s).subset(interior(t))
} by {
    if s.subset(t) {
        forall(x: Real) {
            if interior(s).contains(x) {
                interior_contains_eq(s, x)
                is_interior_point(s, x)
                s.contains(x)
                t.contains(x)
                let eps: Real satisfy {
                    eps.is_positive and forall(y: Real) {
                        y.is_close(x, eps) implies s.contains(y)
                    }
                }
                forall(y: Real) {
                    if y.is_close(x, eps) {
                        s.contains(y)
                        t.contains(y)
                    }
                }
                interior_point_intro(t, x, eps)
                interior(t).contains(x)
            }
        }
    }
}

/// Open sets are equal to their interiors.
theorem open_set_eq_interior(s: Set[Real]) {
    is_open_set(s) implies s = interior(s)
} by {
    if is_open_set(s) {
        forall(x: Real) {
            if s.contains(x) {
                open_set_interior_point(s, x)
                interior_contains_eq(s, x)
                interior(s).contains(x)
            }
        }
        s.subset(interior(s))
        interior_subset(s)
        interior(s).subset(s)
        double_inclusion(s, interior(s))
    }
}

/// The interior of an intersection is contained in the intersection of interiors.
theorem interior_intersection_subset(s: Set[Real], t: Set[Real]) {
    interior(s.intersection(t)).subset(interior(s).intersection(interior(t)))
} by {
    sets_subset_intersection[Real](s, t)
    s.intersection(t).subset(s)
    s.intersection(t).subset(t)
    interior_mono(s.intersection(t), s)
    interior_mono(s.intersection(t), t)
    interior(s.intersection(t)).subset(interior(s))
    interior(s.intersection(t)).subset(interior(t))
    interior(s).superset(interior(s.intersection(t)))
    interior(t).superset(interior(s.intersection(t)))
    set_supset_contains_intersection(interior(s), interior(t), interior(s.intersection(t)))
    interior(s).intersection(interior(t)).superset(interior(s.intersection(t)))
    forall(x: Real) {
        if interior(s.intersection(t)).contains(x) {
            interior(s).contains(x)
            interior(t).contains(x)
            intersection_contains_intro(interior(s), interior(t), x)
        }
    }
    interior(s.intersection(t)).subset(interior(s).intersection(interior(t)))
}

/// The intersection of interiors is contained in the interior of the intersection.
theorem intersection_interior_subset_interior_intersection(s: Set[Real], t: Set[Real]) {
    interior(s).intersection(interior(t)).subset(interior(s.intersection(t)))
} by {
    forall(x: Real) {
        if interior(s).intersection(interior(t)).contains(x) {
            interior(s).contains(x)
            interior(t).contains(x)
            interior_contains_eq(s, x)
            interior_contains_eq(t, x)
            is_interior_point(s, x)
            is_interior_point(t, x)
            interior_point_intersection(s, t, x)
            interior_contains_eq(s.intersection(t), x)
            interior(s.intersection(t)).contains(x)
        }
    }
}

/// Interior distributes over binary intersection.
theorem interior_intersection_eq(s: Set[Real], t: Set[Real]) {
    interior(s.intersection(t)) = interior(s).intersection(interior(t))
} by {
    interior_intersection_subset(s, t)
    intersection_interior_subset_interior_intersection(s, t)
    double_inclusion(interior(s.intersection(t)), interior(s).intersection(interior(t)))
}

/// The union of interiors is contained in the interior of the union.
theorem union_interior_subset_interior_union(s: Set[Real], t: Set[Real]) {
    interior(s).union(interior(t)).subset(interior(s.union(t)))
} by {
    sets_subset_union[Real](s, t)
    s.subset(s.union(t))
    t.subset(s.union(t))
    interior_mono(s, s.union(t))
    interior_mono(t, s.union(t))
    interior(s).subset(interior(s.union(t)))
    interior(t).subset(interior(s.union(t)))
    sets_subset_contain_union[Real](interior(s), interior(t), interior(s.union(t)))
}
