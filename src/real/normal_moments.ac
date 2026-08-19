/// The symmetry and moments of the standard normal law.
///
/// This file develops the standard normal density with the implementation
/// real type of this package (real.real_field.Real), mirroring the facade
/// declarations of real.normal_distribution whose real type lives on the
/// package interface.  The density is
///
///     φ(x) = (-x²/2).exp / (2 π).sqrt,
///
/// and the results here are:
///   - the evenness φ(-x) = φ(x),
///   - the bound φ(x) ≤ 1 / (2 π).sqrt, attained at x = 0,
///   - the derivative facts d/dx [-φ] = x·φ and d/dx [x·φ] = (1 - x²)·φ,
///   - the first moment over a symmetric interval:
///
///         ∫_{-b}^{b} x·φ(x) dx = 0,
///
///     the finite-interval form of E[X] = 0 for the standard normal law.
///
/// The Gaussian integral ∫_{-∞}^{∞} φ = 1 and the CDF reflection
/// Φ(-x) = 1 - Φ(x) remain out of scope: they need an improper-integral
/// notion and the value of the Gaussian integral itself; see the notes in
/// real.normal_distribution and real/gamma.ac.

from real.real_field import Real
from real.exp import exp_pos, exp_zero, two, two_positive
from real.sqrt import sqrt_mul_self
from real.sqrt_inequalities import sqrt_value_nonneg
from real.pi import pi, pi_pos
from real.integral_exp import exp_lte_mono, exp_continuous
from real.gamma import neg_fn
from real.continuity_base import continuous
from real.continuity_sequences import identity_function_is_continuous
from real.continuity_square import square_real
from real.continuity_const_mul import const_mul_left, continuous_const_mul_left
from real.continuity_composition import continuous_compose
from real.continuity_pointwise import continuous_pointwise_neg
from real.continuity_pointwise_mul import continuous_pointwise_mul
from real.calculus_api import is_derivative_fn, derivative_fn_identity, derivative_fn_const_mul, derivative_fn_neg, derivative_fn_mul, derivative_fn_compose
from real.integral import integral, is_integrable, interval_contains, interval_contains_left, interval_contains_right
from real.integral_exp import ftc2_general
from real.integral_trig import fn_integrable_gen
from real.distribution_common import fn_integrable_gen_derivative_bound, integral_ftc2_derivative_bound, lte_of_abs_le, neg_lte_of_abs_le, abs_le_of_lte_and_neg_lte, half_sq_id_continuous, half_sq_id_derivative, is_derivative_fn_both_eq, mul_nonneg_lte, mul_le_mul_nonneg_lte
from data.basic.functions import identity_fn, compose, function_extensionality, function_eq_transport_predicate_rev
from data.basic.function_algebra import pointwise_mul, pointwise_neg, pointwise_add
from data.basic.logic import eq_true_intro
from order import lt_imp_lte, lte_trans, lt_imp_ne, not_lte_imp_gt
from ordered_field import mul_le_mul_of_nonneg_right, zero_is_smaller_than_one, inverse_of_positive_is_positive, mul_le_mul_of_nonpos_right
from algebra.add_ordered_group import add_le_add_right, add_le_add, neg_le_neg
from real.real_ring import mul_abs, square_nonneg, mul_nonneg, mul_le_mul_nonneg, real_mul_comm, mul_pos_pos
from algebra.ring.ring import mul_neg_neg
from real.derivative_exp_log import half_pos
from real.real_base import abs_gte_zero, abs_neg, gt_zero_imp_pos, pos_gt_zero
from real.derivative_trig import abs_of_nonneg
from real.real_series import triangle_ineq

numerals Real

// ---------------------------------------------------------------------------
// The normalizing constant
// ---------------------------------------------------------------------------

/// Two pi is positive.
lemma two_pi_pos {
    two * pi > Real.0
} by {
    two_positive
    two > Real.0
    pi_pos
    pi > Real.0
    mul_pos_pos(two, pi)
    (two * pi).is_positive
    pos_gt_zero(two * pi)
    two * pi > Real.0
}

/// The square root of two pi exists.
theorem normalizing_const_exists {
    exists(y: Real) {
        (two * pi).sqrt = Option.some(y) and y > Real.0
    }
} by {
    two_pi_pos
    two * pi > Real.0
    lt_imp_lte(Real.0, two * pi)
    Real.0 <= two * pi
    sqrt_mul_self(two * pi)
    let y: Real satisfy {
        (two * pi).sqrt = Option.some(y) and y * y = two * pi
    }
    (two * pi).sqrt = Option.some(y)
    y * y = two * pi
    sqrt_value_nonneg(two * pi, y)
    y >= Real.0
    y * y = two * pi
    two * pi != Real.0
    y * y != Real.0
    if y = Real.0 {
        y * y = Real.0 * Real.0
        Real.0 * Real.0 = Real.0
        y * y = Real.0
        false
    }
    y != Real.0
    Real.0 <= y and Real.0 != y
    not_lte_imp_gt(Real.0, y)
    Real.0 < y
    exists(witness: Real) {
        (two * pi).sqrt = Option.some(witness) and witness > Real.0
    }
}

/// The normalizing constant of the standard normal density: (2 pi).sqrt.
let normalizing_const: Real satisfy {
    (two * pi).sqrt = Option.some(normalizing_const)
}

/// The normalizing constant is positive.
theorem normalizing_const_pos {
    normalizing_const > Real.0
} by {
    two_pi_pos
    two * pi > Real.0
    lt_imp_lte(Real.0, two * pi)
    Real.0 <= two * pi
    sqrt_mul_self(two * pi)
    let y: Real satisfy {
        (two * pi).sqrt = Option.some(y) and y * y = two * pi
    }
    (two * pi).sqrt = Option.some(y)
    (two * pi).sqrt = Option.some(normalizing_const)
    some_injective[Real](y, normalizing_const)
    y = normalizing_const
    y * y = two * pi
    normalizing_const * normalizing_const = two * pi
    sqrt_value_nonneg(two * pi, normalizing_const)
    normalizing_const >= Real.0
    two * pi != Real.0
    normalizing_const * normalizing_const != Real.0
    if normalizing_const = Real.0 {
        normalizing_const * normalizing_const = Real.0 * Real.0
        Real.0 * Real.0 = Real.0
        normalizing_const * normalizing_const = Real.0
        false
    }
    normalizing_const != Real.0
    Real.0 <= normalizing_const and Real.0 != normalizing_const
    not_lte_imp_gt(Real.0, normalizing_const)
    Real.0 < normalizing_const
}

/// The reciprocal of the normalizing constant is positive.
lemma normalizing_const_inv_pos {
    Real.1 / normalizing_const > Real.0
} by {
    normalizing_const_pos
    normalizing_const > Real.0
    gt_zero_imp_pos(normalizing_const)
    normalizing_const.is_positive
    inverse_of_positive_is_positive(normalizing_const)
    Real.0 < normalizing_const.inverse
    gt_zero_imp_pos(normalizing_const.inverse)
    normalizing_const.inverse.is_positive
    Real.1.is_positive
    (Real.1 * normalizing_const.inverse).is_positive
    Real.1 / normalizing_const = Real.1 * normalizing_const.inverse
    (Real.1 / normalizing_const).is_positive
    pos_gt_zero(Real.1 / normalizing_const)
    Real.1 / normalizing_const > Real.0
}

/// The reciprocal of the normalizing constant is nonnegative.
lemma normalizing_const_inv_nonneg {
    Real.0 <= Real.1 / normalizing_const
} by {
    normalizing_const_inv_pos
    Real.1 / normalizing_const > Real.0
    lt_imp_lte(Real.0, Real.1 / normalizing_const)
    Real.0 <= Real.1 / normalizing_const
}

// ---------------------------------------------------------------------------
// The density
// ---------------------------------------------------------------------------

/// The standard normal density: φ(x) = (-x²/2).exp / (2 pi).sqrt.
define normal_pdf(x: Real) -> Real {
    (-(x * x) * Real.one_half).exp * (Real.1 / normalizing_const)
}

/// The standard normal density is strictly positive.
theorem normal_pdf_pos(x: Real) {
    normal_pdf(x) > Real.0
} by {
    exp_pos(-(x * x) * Real.one_half)
    (-(x * x) * Real.one_half).exp > Real.0
    gt_zero_imp_pos((-(x * x) * Real.one_half).exp)
    (-(x * x) * Real.one_half).exp.is_positive
    normalizing_const_inv_pos
    Real.1 / normalizing_const > Real.0
    gt_zero_imp_pos(Real.1 / normalizing_const)
    (Real.1 / normalizing_const).is_positive
    mul_pos_pos((-(x * x) * Real.one_half).exp, Real.1 / normalizing_const)
    ((-(x * x) * Real.one_half).exp * (Real.1 / normalizing_const)).is_positive
    pos_gt_zero((-(x * x) * Real.one_half).exp * (Real.1 / normalizing_const))
    (-(x * x) * Real.one_half).exp * (Real.1 / normalizing_const) > Real.0
    normal_pdf(x) = (-(x * x) * Real.one_half).exp * (Real.1 / normalizing_const)
    normal_pdf(x) > Real.0
}

/// The standard normal density is nonnegative.
theorem normal_pdf_nonneg(x: Real) {
    Real.0 <= normal_pdf(x)
} by {
    normal_pdf_pos(x)
    normal_pdf(x) > Real.0
    lt_imp_lte(Real.0, normal_pdf(x))
    Real.0 <= normal_pdf(x)
}

/// The standard normal density is even: φ(-x) = φ(x).
theorem normal_pdf_even(x: Real) {
    normal_pdf(-x) = normal_pdf(x)
} by {
    normal_pdf(-x) = (-((-x) * (-x)) * Real.one_half).exp * (Real.1 / normalizing_const)
    mul_neg_neg(x, x)
    (-x) * (-x) = x * x
    (-((-x) * (-x)) * Real.one_half).exp = (-(x * x) * Real.one_half).exp
    normal_pdf(-x) = (-(x * x) * Real.one_half).exp * (Real.1 / normalizing_const)
    normal_pdf(x) = (-(x * x) * Real.one_half).exp * (Real.1 / normalizing_const)
    normal_pdf(-x) = normal_pdf(x)
}

/// The standard normal density at zero is 1 / (2 pi).sqrt.
theorem normal_pdf_at_zero {
    normal_pdf(Real.0) = Real.1 / normalizing_const
} by {
    normal_pdf(Real.0) = (-(Real.0 * Real.0) * Real.one_half).exp * (Real.1 / normalizing_const)
    Real.0 * Real.0 = Real.0
    -(Real.0 * Real.0) * Real.one_half = Real.0
    (-(Real.0 * Real.0) * Real.one_half).exp = (Real.0).exp
    exp_zero
    (Real.0).exp = Real.1
    normal_pdf(Real.0) = Real.1 * (Real.1 / normalizing_const)
    Real.1 * (Real.1 / normalizing_const) = Real.1 / normalizing_const
    normal_pdf(Real.0) = Real.1 / normalizing_const
}

/// The standard normal density is at most 1 / (2 pi).sqrt.
theorem normal_pdf_upper_bound(x: Real) {
    normal_pdf(x) <= Real.1 / normalizing_const
} by {
    square_nonneg(x)
    x * x >= Real.0
    Real.0 <= x * x
    neg_le_neg(Real.0, x * x)
    -(x * x) <= -Real.0
    -Real.0 = Real.0
    -(x * x) <= Real.0
    half_pos
    Real.one_half > Real.0
    lt_imp_lte(Real.0, Real.one_half)
    Real.0 <= Real.one_half
    mul_le_mul_of_nonneg_right(-(x * x), Real.0, Real.one_half)
    -(x * x) * Real.one_half <= Real.0 * Real.one_half
    Real.0 * Real.one_half = Real.0
    -(x * x) * Real.one_half <= Real.0
    exp_lte_mono(-(x * x) * Real.one_half, Real.0)
    (-(x * x) * Real.one_half).exp <= (Real.0).exp
    exp_zero
    (Real.0).exp = Real.1
    (-(x * x) * Real.one_half).exp <= Real.1
    normalizing_const_inv_nonneg
    Real.0 <= Real.1 / normalizing_const
    mul_le_mul_of_nonneg_right((-(x * x) * Real.one_half).exp, Real.1,
        Real.1 / normalizing_const)
    (-(x * x) * Real.one_half).exp * (Real.1 / normalizing_const) <= Real.1 * (Real.1 / normalizing_const)
    Real.1 * (Real.1 / normalizing_const) = Real.1 / normalizing_const
    (-(x * x) * Real.one_half).exp * (Real.1 / normalizing_const) <= Real.1 / normalizing_const
    normal_pdf(x) = (-(x * x) * Real.one_half).exp * (Real.1 / normalizing_const)
    normal_pdf(x) <= Real.1 / normalizing_const
}

/// The standard normal density attains its maximum at zero.
theorem normal_pdf_max_at_zero(x: Real) {
    normal_pdf(x) <= normal_pdf(Real.0)
} by {
    normal_pdf_upper_bound(x)
    normal_pdf(x) <= Real.1 / normalizing_const
    normal_pdf_at_zero
    normal_pdf(Real.0) = Real.1 / normalizing_const
    normal_pdf(x) <= normal_pdf(Real.0)
}

// ---------------------------------------------------------------------------
// The moment integrand and its antiderivative
// ---------------------------------------------------------------------------

/// The moment integrand x ↦ x·φ(x).
define normal_moment_integrand(x: Real) -> Real {
    x * normal_pdf(x)
}

/// The antiderivative of the moment integrand: x ↦ -φ(x).
define normal_moment_anti(x: Real) -> Real {
    -normal_pdf(x)
}

/// The derivative of the moment integrand: x ↦ (1 - x²)·φ(x).
define normal_moment_deriv(x: Real) -> Real {
    (Real.1 - x * x) * normal_pdf(x)
}

/// The exponent h(x) = -x²/2 is the pointwise negation of x²/2.
define normal_exponent(x: Real) -> Real {
    -(x * x) * Real.one_half
}

/// The exponent function is continuous.
theorem normal_exponent_continuous {
    continuous(normal_exponent)
} by {
    half_sq_id_continuous
    continuous(const_mul_left(Real.one_half,
        pointwise_mul(identity_fn[Real], identity_fn[Real])))
    continuous_pointwise_neg(const_mul_left(Real.one_half,
        pointwise_mul(identity_fn[Real], identity_fn[Real])))
    continuous(pointwise_neg(const_mul_left(Real.one_half,
        pointwise_mul(identity_fn[Real], identity_fn[Real]))))
    forall(x: Real) {
        normal_exponent(x) = -(x * x) * Real.one_half
        const_mul_left(Real.one_half, square_real, x) =
            Real.one_half * square_real(x)
        square_real(x) = x * x
        const_mul_left(Real.one_half, square_real, x) = Real.one_half * (x * x)
        pointwise_neg(const_mul_left(Real.one_half, square_real), x) =
            -const_mul_left(Real.one_half, square_real, x)
        pointwise_neg(const_mul_left(Real.one_half, square_real), x) =
            -(Real.one_half * (x * x))
        Real.one_half * (x * x) = (x * x) * Real.one_half
        -(Real.one_half * (x * x)) = -((x * x) * Real.one_half)
        normal_exponent(x) = pointwise_neg(const_mul_left(Real.one_half, square_real), x)
    }
    function_extensionality(normal_exponent,
        pointwise_neg(const_mul_left(Real.one_half, square_real)))
    define exponent_cont_pred(h: Real -> Real) -> Bool {
        continuous(h)
    }
    exponent_cont_pred(pointwise_neg(const_mul_left(Real.one_half, square_real)))
    function_eq_transport_predicate_rev(exponent_cont_pred, normal_exponent,
        pointwise_neg(const_mul_left(Real.one_half, square_real)))
    continuous(normal_exponent)
}

/// The exponent function has derivative x ↦ -x.
theorem normal_exponent_derivative {
    is_derivative_fn(normal_exponent, neg_fn)
} by {
    half_sq_id_derivative
    is_derivative_fn(const_mul_left(Real.one_half,
        pointwise_mul(identity_fn[Real], identity_fn[Real])), identity_fn[Real])
    derivative_fn_neg(const_mul_left(Real.one_half,
        pointwise_mul(identity_fn[Real], identity_fn[Real])), identity_fn[Real])
    is_derivative_fn(pointwise_neg(const_mul_left(Real.one_half,
        pointwise_mul(identity_fn[Real], identity_fn[Real]))),
        pointwise_neg(identity_fn[Real]))
    forall(x: Real) {
        pointwise_neg(identity_fn[Real], x) = -identity_fn[Real](x)
        identity_fn[Real](x) = x
        pointwise_neg(identity_fn[Real], x) = -x
        neg_fn(x) = -x
        pointwise_neg(identity_fn[Real], x) = neg_fn(x)
    }
    function_extensionality(pointwise_neg(identity_fn[Real]), neg_fn)
    forall(x: Real) {
        normal_exponent(x) = -(x * x) * Real.one_half
        const_mul_left(Real.one_half, square_real, x) =
            Real.one_half * square_real(x)
        square_real(x) = x * x
        const_mul_left(Real.one_half, square_real, x) = Real.one_half * (x * x)
        pointwise_neg(const_mul_left(Real.one_half, square_real), x) =
            -const_mul_left(Real.one_half, square_real, x)
        pointwise_neg(const_mul_left(Real.one_half, square_real), x) =
            -(Real.one_half * (x * x))
        Real.one_half * (x * x) = (x * x) * Real.one_half
        -(Real.one_half * (x * x)) = -((x * x) * Real.one_half)
        normal_exponent(x) = pointwise_neg(const_mul_left(Real.one_half, square_real), x)
    }
    function_extensionality(normal_exponent,
        pointwise_neg(const_mul_left(Real.one_half, square_real)))
    is_derivative_fn_both_eq(
        pointwise_neg(const_mul_left(Real.one_half, square_real)),
        normal_exponent,
        pointwise_neg(identity_fn[Real]),
        neg_fn)
    is_derivative_fn(normal_exponent, neg_fn)
}

/// The standard normal density is continuous.
theorem normal_pdf_continuous {
    continuous(normal_pdf)
} by {
    normal_exponent_continuous
    continuous(normal_exponent)
    exp_continuous
    continuous(Real.exp)
    continuous_compose(normal_exponent, Real.exp)
    continuous(compose(Real.exp, normal_exponent))
    normalizing_const_inv_nonneg
    Real.0 <= Real.1 / normalizing_const
    continuous_const_mul_left(Real.1 / normalizing_const, compose(Real.exp, normal_exponent))
    continuous(const_mul_left(Real.1 / normalizing_const, compose(Real.exp, normal_exponent)))
    forall(x: Real) {
        normal_pdf(x) = (-(x * x) * Real.one_half).exp * (Real.1 / normalizing_const)
        compose(Real.exp, normal_exponent, x) = (normal_exponent(x)).exp
        normal_exponent(x) = -(x * x) * Real.one_half
        compose(Real.exp, normal_exponent, x) = (-(x * x) * Real.one_half).exp
        const_mul_left(Real.1 / normalizing_const, compose(Real.exp, normal_exponent), x) =
            (Real.1 / normalizing_const) * compose(Real.exp, normal_exponent, x)
        (Real.1 / normalizing_const) * (-(x * x) * Real.one_half).exp =
            (-(x * x) * Real.one_half).exp * (Real.1 / normalizing_const)
        const_mul_left(Real.1 / normalizing_const, compose(Real.exp, normal_exponent), x) =
            (-(x * x) * Real.one_half).exp * (Real.1 / normalizing_const)
        normal_pdf(x) = const_mul_left(Real.1 / normalizing_const, compose(Real.exp, normal_exponent), x)
    }
    function_extensionality(normal_pdf,
        const_mul_left(Real.1 / normalizing_const, compose(Real.exp, normal_exponent)))
    define pdf_cont_pred(h: Real -> Real) -> Bool {
        continuous(h)
    }
    pdf_cont_pred(const_mul_left(Real.1 / normalizing_const, compose(Real.exp, normal_exponent)))
    function_eq_transport_predicate_rev(pdf_cont_pred, normal_pdf,
        const_mul_left(Real.1 / normalizing_const, compose(Real.exp, normal_exponent)))
    continuous(normal_pdf)
}

/// The derivative of the normal density is -x·φ(x).
theorem normal_pdf_derivative {
    is_derivative_fn(normal_pdf, pointwise_neg(normal_moment_integrand))
} by {
    normal_exponent_derivative
    is_derivative_fn(normal_exponent, neg_fn)
    derivative_fn_compose(normal_exponent, Real.exp, neg_fn, Real.exp)
    is_derivative_fn(compose(Real.exp, normal_exponent),
        pointwise_mul(compose(Real.exp, normal_exponent), neg_fn))
    derivative_fn_const_mul(Real.1 / normalizing_const, compose(Real.exp, normal_exponent),
        pointwise_mul(compose(Real.exp, normal_exponent), neg_fn))
    is_derivative_fn(
        const_mul_left(Real.1 / normalizing_const, compose(Real.exp, normal_exponent)),
        const_mul_left(Real.1 / normalizing_const,
            pointwise_mul(compose(Real.exp, normal_exponent), neg_fn)))
    forall(x: Real) {
        const_mul_left(Real.1 / normalizing_const,
            pointwise_mul(compose(Real.exp, normal_exponent), neg_fn), x) =
            (Real.1 / normalizing_const) *
                pointwise_mul(compose(Real.exp, normal_exponent), neg_fn, x)
        pointwise_mul(compose(Real.exp, normal_exponent), neg_fn, x) =
            compose(Real.exp, normal_exponent, x) * neg_fn(x)
        compose(Real.exp, normal_exponent, x) = (normal_exponent(x)).exp
        normal_exponent(x) = -(x * x) * Real.one_half
        compose(Real.exp, normal_exponent, x) = (-(x * x) * Real.one_half).exp
        neg_fn(x) = -x
        pointwise_mul(compose(Real.exp, normal_exponent), neg_fn, x) =
            (-(x * x) * Real.one_half).exp * (-x)
        (-(x * x) * Real.one_half).exp * (-x) = -(x * (-(x * x) * Real.one_half).exp)
        const_mul_left(Real.1 / normalizing_const,
            pointwise_mul(compose(Real.exp, normal_exponent), neg_fn), x) =
            (Real.1 / normalizing_const) * (-(x * (-(x * x) * Real.one_half).exp))
        (Real.1 / normalizing_const) * (-(x * (-(x * x) * Real.one_half).exp)) =
            -(x * (-(x * x) * Real.one_half).exp * (Real.1 / normalizing_const))
        normal_pdf(x) = (-(x * x) * Real.one_half).exp * (Real.1 / normalizing_const)
        x * (-(x * x) * Real.one_half).exp * (Real.1 / normalizing_const) =
            x * normal_pdf(x)
        -(x * (-(x * x) * Real.one_half).exp * (Real.1 / normalizing_const)) =
            -(x * normal_pdf(x))
        normal_moment_integrand(x) = x * normal_pdf(x)
        const_mul_left(Real.1 / normalizing_const,
            pointwise_mul(compose(Real.exp, normal_exponent), neg_fn), x) =
            -normal_moment_integrand(x)
        pointwise_neg(normal_moment_integrand, x) = -normal_moment_integrand(x)
        const_mul_left(Real.1 / normalizing_const,
            pointwise_mul(compose(Real.exp, normal_exponent), neg_fn), x) =
            pointwise_neg(normal_moment_integrand, x)
    }
    function_extensionality(
        const_mul_left(Real.1 / normalizing_const,
            pointwise_mul(compose(Real.exp, normal_exponent), neg_fn)),
        pointwise_neg(normal_moment_integrand))
    forall(x: Real) {
        normal_pdf(x) = (-(x * x) * Real.one_half).exp * (Real.1 / normalizing_const)
        compose(Real.exp, normal_exponent, x) = (normal_exponent(x)).exp
        normal_exponent(x) = -(x * x) * Real.one_half
        compose(Real.exp, normal_exponent, x) = (-(x * x) * Real.one_half).exp
        const_mul_left(Real.1 / normalizing_const, compose(Real.exp, normal_exponent), x) =
            (Real.1 / normalizing_const) * compose(Real.exp, normal_exponent, x)
        (Real.1 / normalizing_const) * (-(x * x) * Real.one_half).exp =
            (-(x * x) * Real.one_half).exp * (Real.1 / normalizing_const)
        const_mul_left(Real.1 / normalizing_const, compose(Real.exp, normal_exponent), x) =
            (-(x * x) * Real.one_half).exp * (Real.1 / normalizing_const)
        normal_pdf(x) = const_mul_left(Real.1 / normalizing_const, compose(Real.exp, normal_exponent), x)
    }
    function_extensionality(normal_pdf,
        const_mul_left(Real.1 / normalizing_const, compose(Real.exp, normal_exponent)))
    is_derivative_fn_both_eq(
        const_mul_left(Real.1 / normalizing_const, compose(Real.exp, normal_exponent)),
        normal_pdf,
        const_mul_left(Real.1 / normalizing_const,
            pointwise_mul(compose(Real.exp, normal_exponent), neg_fn)),
        pointwise_neg(normal_moment_integrand))
    is_derivative_fn(normal_pdf, pointwise_neg(normal_moment_integrand))
}

/// The antiderivative x ↦ -φ(x) of the moment integrand.
theorem normal_moment_anti_derivative {
    is_derivative_fn(normal_moment_anti, normal_moment_integrand)
} by {
    normal_pdf_derivative
    is_derivative_fn(normal_pdf, pointwise_neg(normal_moment_integrand))
    derivative_fn_neg(normal_pdf, pointwise_neg(normal_moment_integrand))
    is_derivative_fn(pointwise_neg(normal_pdf),
        pointwise_neg(pointwise_neg(normal_moment_integrand)))
    forall(x: Real) {
        normal_moment_anti(x) = -normal_pdf(x)
        pointwise_neg(normal_pdf, x) = -normal_pdf(x)
        normal_moment_anti(x) = pointwise_neg(normal_pdf, x)
        pointwise_neg(pointwise_neg(normal_moment_integrand), x) =
            -pointwise_neg(normal_moment_integrand, x)
        pointwise_neg(normal_moment_integrand, x) = -normal_moment_integrand(x)
        -(-normal_moment_integrand(x)) = normal_moment_integrand(x)
        pointwise_neg(pointwise_neg(normal_moment_integrand), x) =
            normal_moment_integrand(x)
    }
    function_extensionality(pointwise_neg(pointwise_neg(normal_moment_integrand)),
        normal_moment_integrand)
    eq_true_intro(is_derivative_fn(pointwise_neg(normal_pdf),
        pointwise_neg(pointwise_neg(normal_moment_integrand))))
    (is_derivative_fn(pointwise_neg(normal_pdf),
        pointwise_neg(pointwise_neg(normal_moment_integrand)))) = true
    function_eq_transport_predicate_rev[Real, Real](
        function(h: Real -> Real) { is_derivative_fn(pointwise_neg(normal_pdf), h) },
        pointwise_neg(pointwise_neg(normal_moment_integrand)), normal_moment_integrand)
    is_derivative_fn(pointwise_neg(normal_pdf), normal_moment_integrand)
    function_extensionality(pointwise_neg(normal_pdf), normal_moment_anti)
    eq_true_intro(is_derivative_fn(pointwise_neg(normal_pdf), normal_moment_integrand))
    (is_derivative_fn(pointwise_neg(normal_pdf), normal_moment_integrand)) = true
    function_eq_transport_predicate_rev[Real, Real](
        function(h: Real -> Real) { is_derivative_fn(h, normal_moment_integrand) },
        normal_moment_anti, pointwise_neg(normal_pdf))
    is_derivative_fn(normal_moment_anti, normal_moment_integrand)
}

/// The moment integrand has derivative x ↦ (1 - x²)·φ(x).
theorem normal_moment_derivative {
    is_derivative_fn(normal_moment_integrand, normal_moment_deriv)
} by {
    derivative_fn_identity
    is_derivative_fn(identity_fn[Real], constant[Real, Real](Real.1))
    normal_pdf_derivative
    is_derivative_fn(normal_pdf, pointwise_neg(normal_moment_integrand))
    derivative_fn_mul(identity_fn[Real], normal_pdf,
        constant[Real, Real](Real.1), pointwise_neg(normal_moment_integrand))
    is_derivative_fn(pointwise_mul(identity_fn[Real], normal_pdf),
        pointwise_add(
            pointwise_mul(identity_fn[Real], pointwise_neg(normal_moment_integrand)),
            pointwise_mul(normal_pdf, constant[Real, Real](Real.1))))
    forall(x: Real) {
        normal_moment_integrand(x) = x * normal_pdf(x)
        pointwise_mul(identity_fn[Real], normal_pdf, x) =
            identity_fn[Real](x) * normal_pdf(x)
        identity_fn[Real](x) = x
        pointwise_mul(identity_fn[Real], normal_pdf, x) = x * normal_pdf(x)
        normal_moment_integrand(x) = pointwise_mul(identity_fn[Real], normal_pdf, x)
        pointwise_mul(identity_fn[Real], pointwise_neg(normal_moment_integrand), x) =
            identity_fn[Real](x) * pointwise_neg(normal_moment_integrand, x)
        identity_fn[Real](x) = x
        pointwise_neg(normal_moment_integrand, x) = -normal_moment_integrand(x)
        pointwise_mul(identity_fn[Real], pointwise_neg(normal_moment_integrand), x) =
            x * (-normal_moment_integrand(x))
        x * (-normal_moment_integrand(x)) = -(x * normal_moment_integrand(x))
        normal_moment_integrand(x) = x * normal_pdf(x)
        x * normal_moment_integrand(x) = x * (x * normal_pdf(x))
        x * (x * normal_pdf(x)) = (x * x) * normal_pdf(x)
        -(x * normal_moment_integrand(x)) = -((x * x) * normal_pdf(x))
        pointwise_mul(identity_fn[Real], pointwise_neg(normal_moment_integrand), x) =
            -((x * x) * normal_pdf(x))
        pointwise_mul(normal_pdf, constant[Real, Real](Real.1), x) =
            normal_pdf(x) * constant[Real, Real](Real.1, x)
        constant[Real, Real](Real.1, x) = Real.1
        pointwise_mul(normal_pdf, constant[Real, Real](Real.1), x) = normal_pdf(x)
        pointwise_add(
            pointwise_mul(identity_fn[Real], pointwise_neg(normal_moment_integrand)),
            pointwise_mul(normal_pdf, constant[Real, Real](Real.1)), x) =
            pointwise_mul(identity_fn[Real], pointwise_neg(normal_moment_integrand), x) +
            pointwise_mul(normal_pdf, constant[Real, Real](Real.1), x)
        pointwise_mul(identity_fn[Real], pointwise_neg(normal_moment_integrand), x) +
            pointwise_mul(normal_pdf, constant[Real, Real](Real.1), x) =
            -((x * x) * normal_pdf(x)) + normal_pdf(x)
        pointwise_add(
            pointwise_mul(identity_fn[Real], pointwise_neg(normal_moment_integrand)),
            pointwise_mul(normal_pdf, constant[Real, Real](Real.1)), x) =
            -((x * x) * normal_pdf(x)) + normal_pdf(x)
        normal_pdf(x) - (x * x) * normal_pdf(x) = (Real.1 - x * x) * normal_pdf(x)
        -((x * x) * normal_pdf(x)) + normal_pdf(x) =
            (Real.1 - x * x) * normal_pdf(x)
        normal_moment_deriv(x) = (Real.1 - x * x) * normal_pdf(x)
        pointwise_add(
            pointwise_mul(identity_fn[Real], pointwise_neg(normal_moment_integrand)),
            pointwise_mul(normal_pdf, constant[Real, Real](Real.1)), x) =
            normal_moment_deriv(x)
    }
    function_extensionality(pointwise_mul(identity_fn[Real], normal_pdf),
        normal_moment_integrand)
    function_extensionality(
        pointwise_add(
            pointwise_mul(identity_fn[Real], pointwise_neg(normal_moment_integrand)),
            pointwise_mul(normal_pdf, constant[Real, Real](Real.1))),
        normal_moment_deriv)
    eq_true_intro(is_derivative_fn(pointwise_mul(identity_fn[Real], normal_pdf),
        pointwise_add(
            pointwise_mul(identity_fn[Real], pointwise_neg(normal_moment_integrand)),
            pointwise_mul(normal_pdf, constant[Real, Real](Real.1)))))
    (is_derivative_fn(pointwise_mul(identity_fn[Real], normal_pdf),
        pointwise_add(
            pointwise_mul(identity_fn[Real], pointwise_neg(normal_moment_integrand)),
            pointwise_mul(normal_pdf, constant[Real, Real](Real.1))))) = true
    function_eq_transport_predicate_rev[Real, Real](
        function(h: Real -> Real) { is_derivative_fn(pointwise_mul(identity_fn[Real], normal_pdf), h) },
        pointwise_add(
            pointwise_mul(identity_fn[Real], pointwise_neg(normal_moment_integrand)),
            pointwise_mul(normal_pdf, constant[Real, Real](Real.1))),
        normal_moment_deriv)
    is_derivative_fn(pointwise_mul(identity_fn[Real], normal_pdf), normal_moment_deriv)
    eq_true_intro(is_derivative_fn(pointwise_mul(identity_fn[Real], normal_pdf),
        normal_moment_deriv))
    (is_derivative_fn(pointwise_mul(identity_fn[Real], normal_pdf),
        normal_moment_deriv)) = true
    function_eq_transport_predicate_rev[Real, Real](
        function(h: Real -> Real) { is_derivative_fn(h, normal_moment_deriv) },
        normal_moment_integrand, pointwise_mul(identity_fn[Real], normal_pdf))
    is_derivative_fn(normal_moment_integrand, normal_moment_deriv)
}

/// The moment integrand is continuous.
theorem normal_moment_continuous {
    continuous(normal_moment_integrand)
} by {
    identity_function_is_continuous
    continuous(identity_fn[Real])
    normal_pdf_continuous
    continuous(normal_pdf)
    continuous_pointwise_mul(identity_fn[Real], normal_pdf)
    continuous(pointwise_mul(identity_fn[Real], normal_pdf))
    forall(x: Real) {
        normal_moment_integrand(x) = x * normal_pdf(x)
        pointwise_mul(identity_fn[Real], normal_pdf, x) =
            identity_fn[Real](x) * normal_pdf(x)
        identity_fn[Real](x) = x
        pointwise_mul(identity_fn[Real], normal_pdf, x) = x * normal_pdf(x)
        normal_moment_integrand(x) = pointwise_mul(identity_fn[Real], normal_pdf, x)
    }
    function_extensionality(normal_moment_integrand,
        pointwise_mul(identity_fn[Real], normal_pdf))
    define moment_cont_pred(h: Real -> Real) -> Bool {
        continuous(h)
    }
    moment_cont_pred(pointwise_mul(identity_fn[Real], normal_pdf))
    function_eq_transport_predicate_rev(moment_cont_pred, normal_moment_integrand,
        pointwise_mul(identity_fn[Real], normal_pdf))
    continuous(normal_moment_integrand)
}

/// The antiderivative x ↦ -φ(x) is continuous.
theorem normal_moment_anti_continuous {
    continuous(normal_moment_anti)
} by {
    normal_pdf_continuous
    continuous(normal_pdf)
    continuous_pointwise_neg(normal_pdf)
    continuous(pointwise_neg(normal_pdf))
    forall(x: Real) {
        normal_moment_anti(x) = -normal_pdf(x)
        pointwise_neg(normal_pdf, x) = -normal_pdf(x)
        normal_moment_anti(x) = pointwise_neg(normal_pdf, x)
    }
    function_extensionality(normal_moment_anti, pointwise_neg(normal_pdf))
    define anti_cont_pred(h: Real -> Real) -> Bool {
        continuous(h)
    }
    anti_cont_pred(pointwise_neg(normal_pdf))
    function_eq_transport_predicate_rev(anti_cont_pred, normal_moment_anti,
        pointwise_neg(normal_pdf))
    continuous(normal_moment_anti)
}

// ---------------------------------------------------------------------------
// Bounds on a symmetric interval
// ---------------------------------------------------------------------------

/// The moment integrand is bounded by b/(2 pi).sqrt in absolute value on
/// [-b, b].
theorem normal_moment_bound_on(b: Real) {
    Real.0 <= b implies
    forall(t: Real) {
        interval_contains(-b, b, t) implies
        (normal_moment_integrand(t).abs <= b * (Real.1 / normalizing_const))
    }
} by {
    if Real.0 <= b {
        forall(t: Real) {
            if interval_contains(-b, b, t) {
                normal_moment_integrand(t) = t * normal_pdf(t)
                mul_abs(t, normal_pdf(t))
                (t * normal_pdf(t)).abs = t.abs * normal_pdf(t).abs
                normal_moment_integrand(t).abs = t.abs * normal_pdf(t).abs
                normal_pdf_nonneg(t)
                Real.0 <= normal_pdf(t)
                abs_of_nonneg(normal_pdf(t))
                normal_pdf(t).abs = normal_pdf(t)
                normal_moment_integrand(t).abs = t.abs * normal_pdf(t)
                // |t| <= b on [-b, b]
                interval_contains_right(-b, b, t)
                t <= b
                interval_contains_left(-b, b, t)
                -b <= t
                neg_le_neg(-b, t)
                -t <= -(-b)
                -(-b) = b
                -t <= b
                abs_le_of_lte_and_neg_lte(t, b)
                t.abs <= b
                // φ(t) <= 1/c
                normal_pdf_upper_bound(t)
                normal_pdf(t) <= Real.1 / normalizing_const
                mul_le_mul_nonneg_lte(t.abs, normal_pdf(t), b, Real.1 / normalizing_const)
                t.abs * normal_pdf(t) <= b * (Real.1 / normalizing_const)
                normal_moment_integrand(t).abs <= b * (Real.1 / normalizing_const)
            }
        }
    }
}

/// The moment integrand is bounded below by -b/(2 pi).sqrt on [-b, b].
theorem normal_moment_lower_bound_on(b: Real) {
    Real.0 <= b implies
    forall(t: Real) {
        interval_contains(-b, b, t) implies
        -(b * (Real.1 / normalizing_const)) <= normal_moment_integrand(t)
    }
} by {
    if Real.0 <= b {
        forall(t: Real) {
            if interval_contains(-b, b, t) {
                normal_moment_bound_on(b)
                normal_moment_integrand(t).abs <= b * (Real.1 / normalizing_const)
                neg_lte_of_abs_le(normal_moment_integrand(t),
                    b * (Real.1 / normalizing_const))
                -(b * (Real.1 / normalizing_const)) <= normal_moment_integrand(t)
            }
        }
    }
}

/// The moment integrand is bounded above by b/(2 pi).sqrt on [-b, b].
theorem normal_moment_upper_bound_on(b: Real) {
    Real.0 <= b implies
    forall(t: Real) {
        interval_contains(-b, b, t) implies
        normal_moment_integrand(t) <= b * (Real.1 / normalizing_const)
    }
} by {
    if Real.0 <= b {
        forall(t: Real) {
            if interval_contains(-b, b, t) {
                normal_moment_bound_on(b)
                normal_moment_integrand(t).abs <= b * (Real.1 / normalizing_const)
                lte_of_abs_le(normal_moment_integrand(t),
                    b * (Real.1 / normalizing_const))
                normal_moment_integrand(t) <= b * (Real.1 / normalizing_const)
            }
        }
    }
}

/// The derivative of the moment integrand is bounded on [-b, b].
theorem normal_moment_deriv_bound(b: Real) {
    Real.0 <= b implies
    forall(z: Real) {
        interval_contains(-b, b, z) implies
        normal_moment_deriv(z).abs <= (Real.1 + b * b) * (Real.1 / normalizing_const)
    }
} by {
    if Real.0 <= b {
        forall(z: Real) {
            if interval_contains(-b, b, z) {
                normal_moment_deriv(z) = (Real.1 - z * z) * normal_pdf(z)
                mul_abs(Real.1 - z * z, normal_pdf(z))
                ((Real.1 - z * z) * normal_pdf(z)).abs =
                    (Real.1 - z * z).abs * normal_pdf(z).abs
                normal_moment_deriv(z).abs =
                    (Real.1 - z * z).abs * normal_pdf(z).abs
                normal_pdf_nonneg(z)
                Real.0 <= normal_pdf(z)
                abs_of_nonneg(normal_pdf(z))
                normal_pdf(z).abs = normal_pdf(z)
                normal_moment_deriv(z).abs =
                    (Real.1 - z * z).abs * normal_pdf(z)
                // |1 - z²| <= 1 + z² <= 1 + b²
                triangle_ineq(Real.1, -(z * z))
                (Real.1 + -(z * z)).abs <= Real.1.abs + (-(z * z)).abs
                Real.1 - z * z = Real.1 + -(z * z)
                (Real.1 - z * z).abs <= Real.1.abs + (-(z * z)).abs
                Real.1.abs = Real.1
                abs_neg(z * z)
                (-(z * z)).abs = (z * z).abs
                square_nonneg(z)
                z * z >= Real.0
                Real.0 <= z * z
                abs_of_nonneg(z * z)
                (z * z).abs = z * z
                (-(z * z)).abs = z * z
                Real.1.abs + (-(z * z)).abs = Real.1 + z * z
                (Real.1 - z * z).abs <= Real.1 + z * z
                // |z| <= b gives z² <= b²
                interval_contains_right(-b, b, z)
                z <= b
                interval_contains_left(-b, b, z)
                -b <= z
                neg_le_neg(-b, z)
                -z <= -(-b)
                -(-b) = b
                -z <= b
                abs_le_of_lte_and_neg_lte(z, b)
                z.abs <= b
                mul_abs(z, z)
                (z * z).abs = z.abs * z.abs
                square_nonneg(z)
                z * z >= Real.0
                abs_of_nonneg(z * z)
                (z * z).abs = z * z
                z * z = z.abs * z.abs
                mul_le_mul_nonneg_lte(z.abs, z.abs, b, b)
                z.abs * z.abs <= b * b
                z * z <= b * b
                add_le_add_right(Real.1, Real.1, z * z)
                Real.1 + z * z <= Real.1 + b * b
                lte_trans((Real.1 - z * z).abs, Real.1 + z * z, Real.1 + b * b)
                (Real.1 - z * z).abs <= Real.1 + b * b
                // multiply by 0 <= φ(z) <= 1/c
                abs_gte_zero(Real.1 - z * z)
                Real.0 <= (Real.1 - z * z).abs
                normal_pdf_upper_bound(z)
                normal_pdf(z) <= Real.1 / normalizing_const
                normalizing_const_inv_nonneg
                Real.0 <= Real.1 / normalizing_const
                zero_is_smaller_than_one[Real]
                Real.0 < Real.1
                lt_imp_lte(Real.0, Real.1)
                Real.0 <= Real.1
                square_nonneg(b)
                b * b >= Real.0
                Real.0 <= b * b
                add_le_add(Real.0, Real.1, Real.0, b * b)
                Real.0 + Real.0 <= Real.1 + b * b
                Real.0 + Real.0 = Real.0
                Real.0 <= Real.1 + b * b
                mul_nonneg_lte(Real.1 + b * b, Real.1 / normalizing_const)
                Real.0 <= (Real.1 + b * b) * (Real.1 / normalizing_const)
                mul_le_mul_nonneg_lte((Real.1 - z * z).abs, normal_pdf(z),
                    Real.1 + b * b, Real.1 / normalizing_const)
                (Real.1 - z * z).abs * normal_pdf(z) <= (Real.1 + b * b) * (Real.1 / normalizing_const)
                normal_moment_deriv(z).abs <= (Real.1 + b * b) * (Real.1 / normalizing_const)
            }
        }
    }
}

// ---------------------------------------------------------------------------
// The first moment over a symmetric interval
// ---------------------------------------------------------------------------

/// The first moment of the standard normal law over the symmetric interval
/// [-b, b] is zero.
theorem normal_first_moment_symmetric(b: Real) {
    Real.0 <= b implies
    integral(normal_moment_integrand, -b, b) = Real.0
} by {
    if Real.0 <= b {
        // -b <= b
        neg_le_neg(Real.0, b)
        -b <= -Real.0
        -Real.0 = Real.0
        -b <= Real.0
        lte_trans(-b, Real.0, b)
        -b <= b
        // 0 <= (1 + b²)·(1/c)
        zero_is_smaller_than_one[Real]
        Real.0 < Real.1
        lt_imp_lte(Real.0, Real.1)
        Real.0 <= Real.1
        square_nonneg(b)
        b * b >= Real.0
        Real.0 <= b * b
        add_le_add(Real.0, Real.1, Real.0, b * b)
        Real.0 + Real.0 <= Real.1 + b * b
        Real.0 + Real.0 = Real.0
        Real.0 <= Real.1 + b * b
        normalizing_const_inv_nonneg
        Real.0 <= Real.1 / normalizing_const
        mul_nonneg_lte(Real.1 + b * b, Real.1 / normalizing_const)
        Real.0 <= (Real.1 + b * b) * (Real.1 / normalizing_const)
        normal_moment_deriv_bound(b)
        forall(z: Real) {
            if interval_contains(-b, b, z) {
                normal_moment_deriv_bound(b)
                normal_moment_deriv(z).abs <= (Real.1 + b * b) * (Real.1 / normalizing_const)
            }
        }
        normal_moment_lower_bound_on(b)
        forall(t: Real) {
            if interval_contains(-b, b, t) {
                normal_moment_lower_bound_on(b)
                -(b * (Real.1 / normalizing_const)) <= normal_moment_integrand(t)
            }
        }
        normal_moment_upper_bound_on(b)
        forall(t: Real) {
            if interval_contains(-b, b, t) {
                normal_moment_upper_bound_on(b)
                normal_moment_integrand(t) <= b * (Real.1 / normalizing_const)
            }
        }
        normal_moment_anti_derivative
        is_derivative_fn(normal_moment_anti, normal_moment_integrand)
        normal_moment_derivative
        is_derivative_fn(normal_moment_integrand, normal_moment_deriv)
        normal_moment_continuous
        continuous(normal_moment_integrand)
        normal_moment_anti_continuous
        continuous(normal_moment_anti)
        integral_ftc2_derivative_bound(
            normal_moment_integrand, normal_moment_anti, normal_moment_deriv,
            -b, b, (Real.1 + b * b) * (Real.1 / normalizing_const),
            -(b * (Real.1 / normalizing_const)), b * (Real.1 / normalizing_const))
        is_integrable(normal_moment_integrand, -b, b) and
            integral(normal_moment_integrand, -b, b) =
                normal_moment_anti(b) - normal_moment_anti(-b)
        integral(normal_moment_integrand, -b, b) =
            normal_moment_anti(b) - normal_moment_anti(-b)
        normal_moment_anti(b) = -normal_pdf(b)
        normal_moment_anti(-b) = -normal_pdf(-b)
        normal_pdf_even(b)
        normal_pdf(-b) = normal_pdf(b)
        normal_moment_anti(-b) = -normal_pdf(b)
        normal_moment_anti(b) - normal_moment_anti(-b) =
            -normal_pdf(b) - (-normal_pdf(b))
        -normal_pdf(b) - (-normal_pdf(b)) = Real.0
        integral(normal_moment_integrand, -b, b) = Real.0
    }
}

// ---------------------------------------------------------------------------
// The CDF reflection and the variance (documented)
// ---------------------------------------------------------------------------
//
// The cumulative distribution function Φ(x) = ∫_{-∞}^{x} φ(t) dt and the
// reflection Φ(-x) = 1 - Φ(x) need the Gaussian integral
// ∫_{-∞}^{∞} φ(t) dt = 1, i.e. the value of the improper integral of
// (-t²/2).exp over the whole line.  That requires an improper-integral
// notion and the polar-coordinate evaluation, which the real package does
// not yet provide; see the notes in real.normal_distribution.
//
// The second moment E[X²] = 1 (equivalently Var(X) = 1) also needs the
// Gaussian integral, since the antiderivative of t²·φ(t) is not elementary
// (it involves the error function).  The finite-interval versions above,
// φ(-x) = φ(x) and ∫_{-b}^{b} x·φ(x) dx = 0, are the exact inputs to the
// corresponding limit statements.
