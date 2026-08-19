/// Integrals of polynomial functions over closed intervals, and the
/// extensionality bridge that lets piecewise-defined probability densities be
/// integrated as their polynomial restrictions.
///
/// The main results are:
///   - integral_identity:  integral of the identity over [a, b] is
///     (b^2 - a^2) / 2,
///   - integral_square:    integral of the square over [a, b] is
///     (b^3 - a^3) / 3,
///   - integral_centered_square: integral of (x - m)^2 over [a, b] is
///     ((b - m)^3 - (a - m)^3) / 3,
///   - integral_eq_of_pointwise_eq_on: functions equal on the whole closed
///     interval [a, b] have equal integrals there.
///
/// All value computations go through the fundamental theorem of calculus
/// (ftc2_general from integral_exp.ac): each integrand f is paired with an
/// explicit polynomial antiderivative g, the derivative and continuity of g
/// are transported from the calculus API, and integrability of f follows from
/// the Lipschitz criterion fn_integrable_gen from integral_trig.ac.

from nat import Nat, lt_imp_lte_suc, from_nat
from order import lte_antisymm, lte_trans, lt_imp_lte, lt_trans, lt_imp_ne
from data.basic.functions import identity_fn, function_extensionality, function_eq_transport_predicate_rev
from data.basic.function_algebra import pointwise_mul, pointwise_add
from data.basic.set import Set, set_ext, image_contains, set_image
from data.basic.logic import eq_true_intro
from data.basic.functions import predicate_eq_transport_predicate_rev
from real.real_field import Real
from real.real_base import add_comm, add_assoc, neg_neg, lt_add_right
from real.supremum import is_set_infimum, is_set_supremum, is_set_lower_bound, is_set_upper_bound, is_nonempty, function_image, function_image_contains, function_image_contains_iff, maps_into_set_image, set_supremum_is_upper_bound, set_member_le_supremum, set_supremum_le_upper_bound
from real.integral import integral, is_integrable, interval_contains, interval_set, interval_image, interval_inf, interval_sup, interval_inf_spec, interval_sup_spec, has_lower_bound, has_upper_bound, set_infimum_is_lower_bound, set_lower_bound_contains_le, set_lower_bound_le_infimum, set_infimum_unique, set_supremum_unique, is_partition, partition_mono, partition_point_in_interval, partition_step_lower, partition_step_upper, lower_sum, upper_sum, lower_sum_set, upper_sum_set, lower_sum_contains, upper_sum_contains, diff_step, integral_spec, image_lower_bound, interval_contains_mono, interval_contains_left, interval_contains_right, sub_nonneg, interval_set_contains_left
from real.calculus_api import is_derivative_fn, is_derivative_fn_at, is_derivative_fn_iff, derivative_fn_identity, derivative_fn_square, derivative_fn_const_mul, derivative_fn_mul, derivative_fn_add, derivative_fn_compose
from real.derivative_basic import has_derivative_at
from real.continuity_base import continuous
from real.continuity_const_mul import const_mul_left, continuous_const_mul_left, pointwise_mul_constant_left_eq
from real.continuity_square import square_real, continuous_square_real, square_real_eq_pointwise_mul_identity
from real.continuity_cube import cube_real, cube_real_eq_pointwise_mul_square_identity
from real.continuity_pointwise import continuous_pointwise_add, continuous_pointwise_neg
from real.exp import two, three, two_positive, two_nonzero, mul_one_over, mul_frac_assoc, one_half_plus_one_half
from ordered_field import zero_is_smaller_than_one
from real.harmonic import from_nat_suc_pos_real
from list import partial, partial_pointwise_eq
from real.integral_exp import ftc2_general
from real.integral_trig import fn_integrable_gen
from real.derivative_continuity import div_mul_cancel_denominator
from real.real_ring import mul_assoc, mul_pos_pos, real_mul_comm
from real.real_field import mul_left_cancel

numerals Real
numerals Nat

// ---------------------------------------------------------------------------
// Extensionality of the integral for functions equal on the whole interval
// ---------------------------------------------------------------------------

/// Two functions equal on [x, y] have equal images of that interval.
lemma interval_image_eq_of_pointwise_eq_on(f: Real -> Real, g: Real -> Real, x: Real, y: Real) {
    (forall(t: Real) { interval_contains(x, y, t) implies f(t) = g(t) })
    implies
    interval_image(f, x, y) = interval_image(g, x, y)
} by {
    if forall(t: Real) { interval_contains(x, y, t) implies f(t) = g(t) } {
        forall(v: Real) {
            if interval_image(f, x, y).contains(v) {
                interval_image(f, x, y).contains(v) = function_image_contains(f, interval_set(x, y), v)
                function_image_contains(f, interval_set(x, y), v)
                function_image_contains(f, interval_set(x, y), v) = image_contains(f, interval_set(x, y), v)
                image_contains(f, interval_set(x, y), v)
                let t: Real satisfy {
                    interval_set(x, y).contains(t) and v = f(t)
                }
                interval_set(x, y).contains(t) = interval_contains(x, y, t)
                interval_contains(x, y, t)
                forall(t0: Real) {
                    interval_contains(x, y, t0) implies f(t0) = g(t0)
                }
                interval_contains(x, y, t) implies f(t) = g(t)
                f(t) = g(t)
                v = f(t)
                v = g(t)
                maps_into_set_image(interval_set(x, y), g, t)
                interval_set(x, y).contains(t) implies set_image(interval_set(x, y), g).contains(g(t))
                set_image(interval_set(x, y), g).contains(g(t))
                interval_image(g, x, y) = set_image(interval_set(x, y), g)
                interval_image(g, x, y).contains(g(t))
                v = g(t)
                interval_image(g, x, y).contains(v)
                interval_image(g, x, y).contains(v)
            }
        }
        forall(v: Real) {
            interval_image(f, x, y).contains(v) implies interval_image(g, x, y).contains(v)
        }
        forall(v: Real) {
            if interval_image(g, x, y).contains(v) {
                interval_image(g, x, y).contains(v) = function_image_contains(g, interval_set(x, y), v)
                function_image_contains(g, interval_set(x, y), v)
                function_image_contains(g, interval_set(x, y), v) = image_contains(g, interval_set(x, y), v)
                image_contains(g, interval_set(x, y), v)
                let t: Real satisfy {
                    interval_set(x, y).contains(t) and v = g(t)
                }
                interval_set(x, y).contains(t) = interval_contains(x, y, t)
                interval_contains(x, y, t)
                forall(t0: Real) {
                    interval_contains(x, y, t0) implies f(t0) = g(t0)
                }
                interval_contains(x, y, t) implies f(t) = g(t)
                f(t) = g(t)
                v = g(t)
                v = f(t)
                maps_into_set_image(interval_set(x, y), f, t)
                interval_set(x, y).contains(t) implies set_image(interval_set(x, y), f).contains(f(t))
                set_image(interval_set(x, y), f).contains(f(t))
                interval_image(f, x, y) = set_image(interval_set(x, y), f)
                interval_image(f, x, y).contains(f(t))
                v = f(t)
                interval_image(f, x, y).contains(v)
                interval_image(f, x, y).contains(v)
            }
        }
        forall(v: Real) {
            if interval_image(f, x, y).contains(v) {
                forall(v0: Real) {
                    interval_image(f, x, y).contains(v0) implies interval_image(g, x, y).contains(v0)
                }
                interval_image(f, x, y).contains(v) implies interval_image(g, x, y).contains(v)
                interval_image(g, x, y).contains(v)
                interval_image(f, x, y).contains(v) = interval_image(g, x, y).contains(v)
            }
            if not interval_image(f, x, y).contains(v) {
                forall(v0: Real) {
                    interval_image(g, x, y).contains(v0) implies interval_image(f, x, y).contains(v0)
                }
                interval_image(g, x, y).contains(v) implies interval_image(f, x, y).contains(v)
                if interval_image(g, x, y).contains(v) {
                    interval_image(f, x, y).contains(v)
                    false
                }
                not interval_image(g, x, y).contains(v)
                interval_image(f, x, y).contains(v) = interval_image(g, x, y).contains(v)
            }
            interval_image(f, x, y).contains(v) or not interval_image(f, x, y).contains(v)
            interval_image(f, x, y).contains(v) = interval_image(g, x, y).contains(v)
        }
        set_ext(interval_image(f, x, y), interval_image(g, x, y))
        interval_image(f, x, y) = interval_image(g, x, y)
    }
}

/// A lower bound of f on [x, y] transfers to the image of f on [x, y].
lemma interval_image_lower_bound_of(f: Real -> Real, x: Real, y: Real, lb: Real) {
    (forall(t: Real) { interval_contains(x, y, t) implies lb <= f(t) })
    implies
    has_lower_bound(interval_image(f, x, y))
} by {
    if forall(t: Real) { interval_contains(x, y, t) implies lb <= f(t) } {
        image_lower_bound(f, x, y, lb)
        is_set_lower_bound(interval_image(f, x, y), lb)
        exists(b: Real) {
            is_set_lower_bound(interval_image(f, x, y), b)
        }
        has_lower_bound(interval_image(f, x, y))
    }
}

/// An upper bound of f on [x, y] transfers to the image of f on [x, y].
lemma interval_image_upper_bound_of(f: Real -> Real, x: Real, y: Real, ub: Real) {
    (forall(t: Real) { interval_contains(x, y, t) implies f(t) <= ub })
    implies
    has_upper_bound(interval_image(f, x, y))
} by {
    if forall(t: Real) { interval_contains(x, y, t) implies f(t) <= ub } {
        forall(v: Real) {
            if interval_image(f, x, y).contains(v) {
                interval_image(f, x, y).contains(v) = function_image_contains(f, interval_set(x, y), v)
                function_image_contains(f, interval_set(x, y), v)
                function_image_contains(f, interval_set(x, y), v) = image_contains(f, interval_set(x, y), v)
                image_contains(f, interval_set(x, y), v)
                let t: Real satisfy {
                    interval_set(x, y).contains(t) and v = f(t)
                }
                interval_set(x, y).contains(t) = interval_contains(x, y, t)
                interval_contains(x, y, t)
                forall(t0: Real) {
                    interval_contains(x, y, t0) implies f(t0) <= ub
                }
                interval_contains(x, y, t) implies f(t) <= ub
                f(t) <= ub
                v = f(t)
                v <= ub
            }
        }
        is_set_upper_bound(interval_image(f, x, y), ub)
        exists(b: Real) {
            is_set_upper_bound(interval_image(f, x, y), b)
        }
        has_upper_bound(interval_image(f, x, y))
    }
}

/// The interval infima of functions equal on [x, y] agree, when f is bounded
/// below on [x, y].
theorem interval_inf_eq_of_pointwise_eq_on(f: Real -> Real, g: Real -> Real, x: Real, y: Real, lb: Real) {
    x <= y and
    (forall(t: Real) { interval_contains(x, y, t) implies f(t) = g(t) }) and
    (forall(t: Real) { interval_contains(x, y, t) implies lb <= f(t) })
    implies
    interval_inf(f, x, y) = interval_inf(g, x, y)
} by {
    if x <= y and
       (forall(t: Real) { interval_contains(x, y, t) implies f(t) = g(t) }) and
       (forall(t: Real) { interval_contains(x, y, t) implies lb <= f(t) }) {
        interval_set_contains_left(x, y)
        is_nonempty(interval_set(x, y))
        forall(t: Real) {
            if interval_contains(x, y, t) {
                forall(t0: Real) {
                    interval_contains(x, y, t0) implies lb <= f(t0)
                }
                interval_contains(x, y, t) implies lb <= f(t)
                lb <= f(t)
                forall(t1: Real) {
                    interval_contains(x, y, t1) implies f(t1) = g(t1)
                }
                interval_contains(x, y, t) implies f(t) = g(t)
                f(t) = g(t)
                lb <= g(t)
            }
        }
        forall(t: Real) {
            interval_contains(x, y, t) implies lb <= g(t)
        }
        interval_image_lower_bound_of(g, x, y, lb)
        has_lower_bound(interval_image(g, x, y))
        is_nonempty(interval_set(x, y)) and has_lower_bound(interval_image(g, x, y))
        interval_inf_spec(g, x, y)
        is_set_infimum(interval_image(g, x, y), interval_inf(g, x, y))
        set_infimum_is_lower_bound(interval_image(g, x, y), interval_inf(g, x, y))
        is_set_lower_bound(interval_image(g, x, y), interval_inf(g, x, y))
        // interval_inf(g, x, y) is a lower bound of the image of f.
        forall(v: Real) {
            if interval_image(f, x, y).contains(v) {
                interval_image(f, x, y).contains(v) = function_image_contains(f, interval_set(x, y), v)
                function_image_contains(f, interval_set(x, y), v)
                function_image_contains(f, interval_set(x, y), v) = image_contains(f, interval_set(x, y), v)
                image_contains(f, interval_set(x, y), v)
                let t: Real satisfy {
                    interval_set(x, y).contains(t) and v = f(t)
                }
                interval_set(x, y).contains(t) = interval_contains(x, y, t)
                interval_contains(x, y, t)
                forall(t0: Real) {
                    interval_contains(x, y, t0) implies f(t0) = g(t0)
                }
                interval_contains(x, y, t) implies f(t) = g(t)
                f(t) = g(t)
                v = f(t)
                v = g(t)
                maps_into_set_image(interval_set(x, y), g, t)
                set_image(interval_set(x, y), g).contains(g(t))
                interval_image(g, x, y) = set_image(interval_set(x, y), g)
                interval_image(g, x, y).contains(g(t))
                interval_image(g, x, y).contains(g(t))
                set_lower_bound_contains_le(interval_image(g, x, y), interval_inf(g, x, y), g(t))
                interval_inf(g, x, y) <= g(t)
                v = g(t)
                interval_inf(g, x, y) <= v
            }
        }
        is_set_lower_bound(interval_image(f, x, y), interval_inf(g, x, y))
        // Every lower bound of the image of f is a lower bound of the image of g.
        forall(b: Real) {
            if is_set_lower_bound(interval_image(f, x, y), b) {
                forall(v: Real) {
                    if interval_image(g, x, y).contains(v) {
                        interval_image(g, x, y).contains(v) = function_image_contains(g, interval_set(x, y), v)
                        function_image_contains(g, interval_set(x, y), v)
                        function_image_contains(g, interval_set(x, y), v) = image_contains(g, interval_set(x, y), v)
                        image_contains(g, interval_set(x, y), v)
                        let t: Real satisfy {
                            interval_set(x, y).contains(t) and v = g(t)
                        }
                        interval_set(x, y).contains(t) = interval_contains(x, y, t)
                        interval_contains(x, y, t)
                        forall(t0: Real) {
                            interval_contains(x, y, t0) implies f(t0) = g(t0)
                        }
                        interval_contains(x, y, t) implies f(t) = g(t)
                        f(t) = g(t)
                        v = g(t)
                        v = f(t)
                        maps_into_set_image(interval_set(x, y), f, t)
                        set_image(interval_set(x, y), f).contains(f(t))
                        interval_image(f, x, y) = set_image(interval_set(x, y), f)
                        interval_image(f, x, y).contains(f(t))
                        interval_image(f, x, y).contains(f(t))
                        is_set_lower_bound(interval_image(f, x, y), b) = forall(w: Real) {
                            interval_image(f, x, y).contains(w) implies b <= w
                        }
                        forall(w: Real) {
                            interval_image(f, x, y).contains(w) implies b <= w
                        }
                        interval_image(f, x, y).contains(f(t)) implies b <= f(t)
                        b <= f(t)
                        v = f(t)
                        b <= v
                    }
                }
                is_set_lower_bound(interval_image(g, x, y), b)
                set_lower_bound_le_infimum(interval_image(g, x, y), interval_inf(g, x, y), b)
                b <= interval_inf(g, x, y)
            }
        }
        is_set_lower_bound(interval_image(f, x, y), interval_inf(g, x, y)) and forall(b: Real) {
            is_set_lower_bound(interval_image(f, x, y), b) implies b <= interval_inf(g, x, y)
        }
        is_set_infimum(interval_image(f, x, y), interval_inf(g, x, y))
        interval_image_lower_bound_of(f, x, y, lb)
        has_lower_bound(interval_image(f, x, y))
        is_nonempty(interval_set(x, y)) and has_lower_bound(interval_image(f, x, y))
        interval_inf_spec(f, x, y)
        is_set_infimum(interval_image(f, x, y), interval_inf(f, x, y))
        set_infimum_unique(interval_image(f, x, y), interval_inf(f, x, y), interval_inf(g, x, y))
        interval_inf(f, x, y) = interval_inf(g, x, y)
    }
}

/// The interval suprema of functions equal on [x, y] agree, when f is bounded
/// above on [x, y].
theorem interval_sup_eq_of_pointwise_eq_on(f: Real -> Real, g: Real -> Real, x: Real, y: Real, ub: Real) {
    x <= y and
    (forall(t: Real) { interval_contains(x, y, t) implies f(t) = g(t) }) and
    (forall(t: Real) { interval_contains(x, y, t) implies f(t) <= ub })
    implies
    interval_sup(f, x, y) = interval_sup(g, x, y)
} by {
    if x <= y and
       (forall(t: Real) { interval_contains(x, y, t) implies f(t) = g(t) }) and
       (forall(t: Real) { interval_contains(x, y, t) implies f(t) <= ub }) {
        interval_set_contains_left(x, y)
        is_nonempty(interval_set(x, y))
        forall(t: Real) {
            if interval_contains(x, y, t) {
                forall(t0: Real) {
                    interval_contains(x, y, t0) implies f(t0) <= ub
                }
                interval_contains(x, y, t) implies f(t) <= ub
                f(t) <= ub
                forall(t1: Real) {
                    interval_contains(x, y, t1) implies f(t1) = g(t1)
                }
                interval_contains(x, y, t) implies f(t) = g(t)
                f(t) = g(t)
                g(t) <= ub
            }
        }
        forall(t: Real) {
            interval_contains(x, y, t) implies g(t) <= ub
        }
        interval_image_upper_bound_of(g, x, y, ub)
        has_upper_bound(interval_image(g, x, y))
        is_nonempty(interval_set(x, y)) and has_upper_bound(interval_image(g, x, y))
        interval_sup_spec(g, x, y)
        is_set_supremum(interval_image(g, x, y), interval_sup(g, x, y))
        set_supremum_is_upper_bound(interval_image(g, x, y), interval_sup(g, x, y))
        is_set_upper_bound(interval_image(g, x, y), interval_sup(g, x, y))
        // interval_sup(g, x, y) is an upper bound of the image of f.
        forall(v: Real) {
            if interval_image(f, x, y).contains(v) {
                interval_image(f, x, y).contains(v) = function_image_contains(f, interval_set(x, y), v)
                function_image_contains(f, interval_set(x, y), v)
                function_image_contains(f, interval_set(x, y), v) = image_contains(f, interval_set(x, y), v)
                image_contains(f, interval_set(x, y), v)
                let t: Real satisfy {
                    interval_set(x, y).contains(t) and v = f(t)
                }
                interval_set(x, y).contains(t) = interval_contains(x, y, t)
                interval_contains(x, y, t)
                forall(t0: Real) {
                    interval_contains(x, y, t0) implies f(t0) = g(t0)
                }
                interval_contains(x, y, t) implies f(t) = g(t)
                f(t) = g(t)
                v = f(t)
                v = g(t)
                maps_into_set_image(interval_set(x, y), g, t)
                set_image(interval_set(x, y), g).contains(g(t))
                interval_image(g, x, y) = set_image(interval_set(x, y), g)
                interval_image(g, x, y).contains(g(t))
                interval_image(g, x, y).contains(g(t))
                set_member_le_supremum(interval_image(g, x, y), interval_sup(g, x, y), g(t))
                g(t) <= interval_sup(g, x, y)
                v = g(t)
                v <= interval_sup(g, x, y)
            }
        }
        is_set_upper_bound(interval_image(f, x, y), interval_sup(g, x, y))
        // Every upper bound of the image of f is an upper bound of the image of g.
        forall(b: Real) {
            if is_set_upper_bound(interval_image(f, x, y), b) {
                forall(v: Real) {
                    if interval_image(g, x, y).contains(v) {
                        interval_image(g, x, y).contains(v) = function_image_contains(g, interval_set(x, y), v)
                        function_image_contains(g, interval_set(x, y), v)
                        function_image_contains(g, interval_set(x, y), v) = image_contains(g, interval_set(x, y), v)
                        image_contains(g, interval_set(x, y), v)
                        let t: Real satisfy {
                            interval_set(x, y).contains(t) and v = g(t)
                        }
                        interval_set(x, y).contains(t) = interval_contains(x, y, t)
                        interval_contains(x, y, t)
                        forall(t0: Real) {
                            interval_contains(x, y, t0) implies f(t0) = g(t0)
                        }
                        interval_contains(x, y, t) implies f(t) = g(t)
                        f(t) = g(t)
                        v = g(t)
                        v = f(t)
                        maps_into_set_image(interval_set(x, y), f, t)
                        set_image(interval_set(x, y), f).contains(f(t))
                        interval_image(f, x, y) = set_image(interval_set(x, y), f)
                        interval_image(f, x, y).contains(f(t))
                        interval_image(f, x, y).contains(f(t))
                        is_set_upper_bound(interval_image(f, x, y), b) = forall(w: Real) {
                            interval_image(f, x, y).contains(w) implies w <= b
                        }
                        forall(w: Real) {
                            interval_image(f, x, y).contains(w) implies w <= b
                        }
                        interval_image(f, x, y).contains(f(t)) implies f(t) <= b
                        f(t) <= b
                        v = f(t)
                        v <= b
                    }
                }
                is_set_upper_bound(interval_image(g, x, y), b)
                set_supremum_le_upper_bound(interval_image(g, x, y), interval_sup(g, x, y), b)
                interval_sup(g, x, y) <= b
            }
        }
        is_set_upper_bound(interval_image(f, x, y), interval_sup(g, x, y)) and forall(b: Real) {
            is_set_upper_bound(interval_image(f, x, y), b) implies interval_sup(g, x, y) <= b
        }
        is_set_supremum(interval_image(f, x, y), interval_sup(g, x, y))
        interval_image_upper_bound_of(f, x, y, ub)
        has_upper_bound(interval_image(f, x, y))
        is_nonempty(interval_set(x, y)) and has_upper_bound(interval_image(f, x, y))
        interval_sup_spec(f, x, y)
        is_set_supremum(interval_image(f, x, y), interval_sup(f, x, y))
        set_supremum_unique(interval_image(f, x, y), interval_sup(f, x, y), interval_sup(g, x, y))
        interval_sup(f, x, y) = interval_sup(g, x, y)
    }
}

/// The lower Darboux steps of functions equal on [a, b] agree on every
/// subinterval of a partition of [a, b].
theorem partition_step_lower_eq_of_pointwise_eq_on(
    f: Real -> Real, g: Real -> Real, p: Nat -> Real, a: Real, b: Real, n: Nat, k: Nat, lb: Real
) {
    is_partition(p, a, b, n) and k < n and
    (forall(t: Real) { interval_contains(a, b, t) implies f(t) = g(t) }) and
    (forall(t: Real) { interval_contains(a, b, t) implies lb <= f(t) })
    implies
    partition_step_lower(f, p, k) = partition_step_lower(g, p, k)
} by {
    if is_partition(p, a, b, n) and k < n and
       (forall(t: Real) { interval_contains(a, b, t) implies f(t) = g(t) }) and
       (forall(t: Real) { interval_contains(a, b, t) implies lb <= f(t) }) {
        lt_imp_lte_suc(k, n)
        k + 1 <= n
        k <= k + 1
        partition_mono(p, a, b, n, k, k + 1)
        p(k) <= p(k + 1)
        partition_point_in_interval(p, a, b, n, k)
        interval_contains(a, b, p(k))
        partition_point_in_interval(p, a, b, n, k + 1)
        interval_contains(a, b, p(k + 1))
        forall(t: Real) {
            if interval_contains(p(k), p(k + 1), t) {
                interval_contains_left(p(k), p(k + 1), t)
                p(k) <= t
                interval_contains_right(p(k), p(k + 1), t)
                t <= p(k + 1)
                interval_contains_mono(a, b, p(k), p(k + 1), t)
                interval_contains(a, b, t)
                forall(t0: Real) {
                    interval_contains(a, b, t0) implies f(t0) = g(t0)
                }
                interval_contains(a, b, t) implies f(t) = g(t)
                f(t) = g(t)
            }
        }
        forall(t: Real) {
            interval_contains(p(k), p(k + 1), t) implies f(t) = g(t)
        }
        forall(t: Real) {
            if interval_contains(p(k), p(k + 1), t) {
                interval_contains_left(p(k), p(k + 1), t)
                p(k) <= t
                interval_contains_right(p(k), p(k + 1), t)
                t <= p(k + 1)
                interval_contains_mono(a, b, p(k), p(k + 1), t)
                interval_contains(a, b, t)
                forall(t1: Real) {
                    interval_contains(a, b, t1) implies lb <= f(t1)
                }
                interval_contains(a, b, t) implies lb <= f(t)
                lb <= f(t)
            }
        }
        forall(t: Real) {
            interval_contains(p(k), p(k + 1), t) implies lb <= f(t)
        }
        eq_true_intro(forall(t: Real) { interval_contains(p(k), p(k + 1), t) implies f(t) = g(t) })
        (forall(t: Real) { interval_contains(p(k), p(k + 1), t) implies f(t) = g(t) }) = true
        eq_true_intro(forall(t: Real) { interval_contains(p(k), p(k + 1), t) implies lb <= f(t) })
        (forall(t: Real) { interval_contains(p(k), p(k + 1), t) implies lb <= f(t) }) = true
        p(k) <= p(k + 1) and
            (forall(t: Real) { interval_contains(p(k), p(k + 1), t) implies f(t) = g(t) }) and
            (forall(t: Real) { interval_contains(p(k), p(k + 1), t) implies lb <= f(t) })
        interval_inf_eq_of_pointwise_eq_on(f, g, p(k), p(k + 1), lb)
        interval_inf(f, p(k), p(k + 1)) = interval_inf(g, p(k), p(k + 1))
        partition_step_lower(f, p, k) = interval_inf(f, p(k), p(k + 1)) * diff_step(p, k)
        partition_step_lower(g, p, k) = interval_inf(g, p(k), p(k + 1)) * diff_step(p, k)
        partition_step_lower(f, p, k) = interval_inf(g, p(k), p(k + 1)) * diff_step(p, k)
        partition_step_lower(f, p, k) = partition_step_lower(g, p, k)
    }
}

/// The upper Darboux steps of functions equal on [a, b] agree on every
/// subinterval of a partition of [a, b].
theorem partition_step_upper_eq_of_pointwise_eq_on(
    f: Real -> Real, g: Real -> Real, p: Nat -> Real, a: Real, b: Real, n: Nat, k: Nat, ub: Real
) {
    is_partition(p, a, b, n) and k < n and
    (forall(t: Real) { interval_contains(a, b, t) implies f(t) = g(t) }) and
    (forall(t: Real) { interval_contains(a, b, t) implies f(t) <= ub })
    implies
    partition_step_upper(f, p, k) = partition_step_upper(g, p, k)
} by {
    if is_partition(p, a, b, n) and k < n and
       (forall(t: Real) { interval_contains(a, b, t) implies f(t) = g(t) }) and
       (forall(t: Real) { interval_contains(a, b, t) implies f(t) <= ub }) {
        lt_imp_lte_suc(k, n)
        k + 1 <= n
        k <= k + 1
        partition_mono(p, a, b, n, k, k + 1)
        p(k) <= p(k + 1)
        partition_point_in_interval(p, a, b, n, k)
        interval_contains(a, b, p(k))
        partition_point_in_interval(p, a, b, n, k + 1)
        interval_contains(a, b, p(k + 1))
        forall(t: Real) {
            if interval_contains(p(k), p(k + 1), t) {
                interval_contains_left(p(k), p(k + 1), t)
                p(k) <= t
                interval_contains_right(p(k), p(k + 1), t)
                t <= p(k + 1)
                interval_contains_mono(a, b, p(k), p(k + 1), t)
                interval_contains(a, b, t)
                forall(t0: Real) {
                    interval_contains(a, b, t0) implies f(t0) = g(t0)
                }
                interval_contains(a, b, t) implies f(t) = g(t)
                f(t) = g(t)
            }
        }
        forall(t: Real) {
            interval_contains(p(k), p(k + 1), t) implies f(t) = g(t)
        }
        forall(t: Real) {
            if interval_contains(p(k), p(k + 1), t) {
                interval_contains_left(p(k), p(k + 1), t)
                p(k) <= t
                interval_contains_right(p(k), p(k + 1), t)
                t <= p(k + 1)
                interval_contains_mono(a, b, p(k), p(k + 1), t)
                interval_contains(a, b, t)
                forall(t1: Real) {
                    interval_contains(a, b, t1) implies f(t1) <= ub
                }
                interval_contains(a, b, t) implies f(t) <= ub
                f(t) <= ub
            }
        }
        forall(t: Real) {
            interval_contains(p(k), p(k + 1), t) implies f(t) <= ub
        }
        eq_true_intro(forall(t: Real) { interval_contains(p(k), p(k + 1), t) implies f(t) = g(t) })
        (forall(t: Real) { interval_contains(p(k), p(k + 1), t) implies f(t) = g(t) }) = true
        eq_true_intro(forall(t: Real) { interval_contains(p(k), p(k + 1), t) implies f(t) <= ub })
        (forall(t: Real) { interval_contains(p(k), p(k + 1), t) implies f(t) <= ub }) = true
        p(k) <= p(k + 1) and
            (forall(t: Real) { interval_contains(p(k), p(k + 1), t) implies f(t) = g(t) }) and
            (forall(t: Real) { interval_contains(p(k), p(k + 1), t) implies f(t) <= ub })
        interval_sup_eq_of_pointwise_eq_on(f, g, p(k), p(k + 1), ub)
        interval_sup(f, p(k), p(k + 1)) = interval_sup(g, p(k), p(k + 1))
        partition_step_upper(f, p, k) = interval_sup(f, p(k), p(k + 1)) * diff_step(p, k)
        partition_step_upper(g, p, k) = interval_sup(g, p(k), p(k + 1)) * diff_step(p, k)
        partition_step_upper(f, p, k) = interval_sup(g, p(k), p(k + 1)) * diff_step(p, k)
        partition_step_upper(f, p, k) = partition_step_upper(g, p, k)
    }
}

/// The lower Darboux sums of functions equal on [a, b] agree over every
/// partition of [a, b].
theorem lower_sum_eq_of_pointwise_eq_on(f: Real -> Real, g: Real -> Real, p: Nat -> Real, a: Real, b: Real, n: Nat, lb: Real) {
    is_partition(p, a, b, n) and
    (forall(t: Real) { interval_contains(a, b, t) implies f(t) = g(t) }) and
    (forall(t: Real) { interval_contains(a, b, t) implies lb <= f(t) })
    implies
    lower_sum(f, p, n) = lower_sum(g, p, n)
} by {
    if is_partition(p, a, b, n) and
       (forall(t: Real) { interval_contains(a, b, t) implies f(t) = g(t) }) and
       (forall(t: Real) { interval_contains(a, b, t) implies lb <= f(t) }) {
        forall(k: Nat) {
            if k < n {
                partition_step_lower_eq_of_pointwise_eq_on(f, g, p, a, b, n, k, lb)
                partition_step_lower(f, p, k) = partition_step_lower(g, p, k)
            }
        }
        forall(k: Nat) {
            k < n implies partition_step_lower(f, p, k) = partition_step_lower(g, p, k)
        }
        partial_pointwise_eq(partition_step_lower(f, p), partition_step_lower(g, p), n)
        partial(partition_step_lower(f, p), n) = partial(partition_step_lower(g, p), n)
        lower_sum(f, p, n) = partial(partition_step_lower(f, p), n)
        lower_sum(g, p, n) = partial(partition_step_lower(g, p), n)
        lower_sum(f, p, n) = lower_sum(g, p, n)
    }
}

/// The upper Darboux sums of functions equal on [a, b] agree over every
/// partition of [a, b].
theorem upper_sum_eq_of_pointwise_eq_on(f: Real -> Real, g: Real -> Real, p: Nat -> Real, a: Real, b: Real, n: Nat, ub: Real) {
    is_partition(p, a, b, n) and
    (forall(t: Real) { interval_contains(a, b, t) implies f(t) = g(t) }) and
    (forall(t: Real) { interval_contains(a, b, t) implies f(t) <= ub })
    implies
    upper_sum(f, p, n) = upper_sum(g, p, n)
} by {
    if is_partition(p, a, b, n) and
       (forall(t: Real) { interval_contains(a, b, t) implies f(t) = g(t) }) and
       (forall(t: Real) { interval_contains(a, b, t) implies f(t) <= ub }) {
        forall(k: Nat) {
            if k < n {
                partition_step_upper_eq_of_pointwise_eq_on(f, g, p, a, b, n, k, ub)
                partition_step_upper(f, p, k) = partition_step_upper(g, p, k)
            }
        }
        forall(k: Nat) {
            k < n implies partition_step_upper(f, p, k) = partition_step_upper(g, p, k)
        }
        partial_pointwise_eq(partition_step_upper(f, p), partition_step_upper(g, p), n)
        partial(partition_step_upper(f, p), n) = partial(partition_step_upper(g, p), n)
        upper_sum(f, p, n) = partial(partition_step_upper(f, p), n)
        upper_sum(g, p, n) = partial(partition_step_upper(g, p), n)
        upper_sum(f, p, n) = upper_sum(g, p, n)
    }
}

/// The sets of lower sums of functions equal on [a, b] agree.
theorem lower_sum_set_eq_of_pointwise_eq_on(f: Real -> Real, g: Real -> Real, a: Real, b: Real, lb: Real) {
    (forall(t: Real) { interval_contains(a, b, t) implies f(t) = g(t) }) and
    (forall(t: Real) { interval_contains(a, b, t) implies lb <= f(t) })
    implies
    lower_sum_set(f, a, b) = lower_sum_set(g, a, b)
} by {
    if (forall(t: Real) { interval_contains(a, b, t) implies f(t) = g(t) }) and
       (forall(t: Real) { interval_contains(a, b, t) implies lb <= f(t) }) {
        forall(s: Real) {
            if lower_sum_set(f, a, b).contains(s) {
                lower_sum_set(f, a, b).contains(s) = lower_sum_contains(f, a, b, s)
                lower_sum_contains(f, a, b, s)
                let (p: Nat -> Real, n: Nat) satisfy {
                    is_partition(p, a, b, n) and s = lower_sum(f, p, n)
                }
                lower_sum_eq_of_pointwise_eq_on(f, g, p, a, b, n, lb)
                lower_sum(f, p, n) = lower_sum(g, p, n)
                is_partition(p, a, b, n) and s = lower_sum(g, p, n)
                lower_sum_contains(g, a, b, s)
                lower_sum_set(g, a, b).contains(s) = lower_sum_contains(g, a, b, s)
                lower_sum_set(g, a, b).contains(s)
            }
        }
        forall(s: Real) {
            lower_sum_set(f, a, b).contains(s) implies lower_sum_set(g, a, b).contains(s)
        }
        forall(s: Real) {
            if lower_sum_set(g, a, b).contains(s) {
                lower_sum_set(g, a, b).contains(s) = lower_sum_contains(g, a, b, s)
                lower_sum_contains(g, a, b, s)
                let (p: Nat -> Real, n: Nat) satisfy {
                    is_partition(p, a, b, n) and s = lower_sum(g, p, n)
                }
                lower_sum_eq_of_pointwise_eq_on(f, g, p, a, b, n, lb)
                lower_sum(f, p, n) = lower_sum(g, p, n)
                is_partition(p, a, b, n) and s = lower_sum(f, p, n)
                lower_sum_contains(f, a, b, s)
                lower_sum_set(f, a, b).contains(s) = lower_sum_contains(f, a, b, s)
                lower_sum_set(f, a, b).contains(s)
            }
        }
        forall(s: Real) {
            lower_sum_set(g, a, b).contains(s) implies lower_sum_set(f, a, b).contains(s)
        }
        forall(s: Real) {
            if lower_sum_set(f, a, b).contains(s) {
                forall(s0: Real) {
                    lower_sum_set(f, a, b).contains(s0) implies lower_sum_set(g, a, b).contains(s0)
                }
                lower_sum_set(f, a, b).contains(s) implies lower_sum_set(g, a, b).contains(s)
                lower_sum_set(g, a, b).contains(s)
                lower_sum_set(f, a, b).contains(s) = lower_sum_set(g, a, b).contains(s)
            }
            if not lower_sum_set(f, a, b).contains(s) {
                forall(s0: Real) {
                    lower_sum_set(g, a, b).contains(s0) implies lower_sum_set(f, a, b).contains(s0)
                }
                lower_sum_set(g, a, b).contains(s) implies lower_sum_set(f, a, b).contains(s)
                if lower_sum_set(g, a, b).contains(s) {
                    lower_sum_set(f, a, b).contains(s)
                    false
                }
                not lower_sum_set(g, a, b).contains(s)
                lower_sum_set(f, a, b).contains(s) = lower_sum_set(g, a, b).contains(s)
            }
            lower_sum_set(f, a, b).contains(s) or not lower_sum_set(f, a, b).contains(s)
            lower_sum_set(f, a, b).contains(s) = lower_sum_set(g, a, b).contains(s)
        }
        set_ext(lower_sum_set(f, a, b), lower_sum_set(g, a, b))
        lower_sum_set(f, a, b) = lower_sum_set(g, a, b)
    }
}

/// The sets of upper sums of functions equal on [a, b] agree.
theorem upper_sum_set_eq_of_pointwise_eq_on(f: Real -> Real, g: Real -> Real, a: Real, b: Real, ub: Real) {
    (forall(t: Real) { interval_contains(a, b, t) implies f(t) = g(t) }) and
    (forall(t: Real) { interval_contains(a, b, t) implies f(t) <= ub })
    implies
    upper_sum_set(f, a, b) = upper_sum_set(g, a, b)
} by {
    if (forall(t: Real) { interval_contains(a, b, t) implies f(t) = g(t) }) and
       (forall(t: Real) { interval_contains(a, b, t) implies f(t) <= ub }) {
        forall(s: Real) {
            if upper_sum_set(f, a, b).contains(s) {
                upper_sum_set(f, a, b).contains(s) = upper_sum_contains(f, a, b, s)
                upper_sum_contains(f, a, b, s)
                let (p: Nat -> Real, n: Nat) satisfy {
                    is_partition(p, a, b, n) and s = upper_sum(f, p, n)
                }
                upper_sum_eq_of_pointwise_eq_on(f, g, p, a, b, n, ub)
                upper_sum(f, p, n) = upper_sum(g, p, n)
                is_partition(p, a, b, n) and s = upper_sum(g, p, n)
                upper_sum_contains(g, a, b, s)
                upper_sum_set(g, a, b).contains(s) = upper_sum_contains(g, a, b, s)
                upper_sum_set(g, a, b).contains(s)
            }
        }
        forall(s: Real) {
            upper_sum_set(f, a, b).contains(s) implies upper_sum_set(g, a, b).contains(s)
        }
        forall(s: Real) {
            if upper_sum_set(g, a, b).contains(s) {
                upper_sum_set(g, a, b).contains(s) = upper_sum_contains(g, a, b, s)
                upper_sum_contains(g, a, b, s)
                let (p: Nat -> Real, n: Nat) satisfy {
                    is_partition(p, a, b, n) and s = upper_sum(g, p, n)
                }
                upper_sum_eq_of_pointwise_eq_on(f, g, p, a, b, n, ub)
                upper_sum(f, p, n) = upper_sum(g, p, n)
                is_partition(p, a, b, n) and s = upper_sum(f, p, n)
                upper_sum_contains(f, a, b, s)
                upper_sum_set(f, a, b).contains(s) = upper_sum_contains(f, a, b, s)
                upper_sum_set(f, a, b).contains(s)
            }
        }
        forall(s: Real) {
            upper_sum_set(g, a, b).contains(s) implies upper_sum_set(f, a, b).contains(s)
        }
        forall(s: Real) {
            if upper_sum_set(f, a, b).contains(s) {
                forall(s0: Real) {
                    upper_sum_set(f, a, b).contains(s0) implies upper_sum_set(g, a, b).contains(s0)
                }
                upper_sum_set(f, a, b).contains(s) implies upper_sum_set(g, a, b).contains(s)
                upper_sum_set(g, a, b).contains(s)
                upper_sum_set(f, a, b).contains(s) = upper_sum_set(g, a, b).contains(s)
            }
            if not upper_sum_set(f, a, b).contains(s) {
                forall(s0: Real) {
                    upper_sum_set(g, a, b).contains(s0) implies upper_sum_set(f, a, b).contains(s0)
                }
                upper_sum_set(g, a, b).contains(s) implies upper_sum_set(f, a, b).contains(s)
                if upper_sum_set(g, a, b).contains(s) {
                    upper_sum_set(f, a, b).contains(s)
                    false
                }
                not upper_sum_set(g, a, b).contains(s)
                upper_sum_set(f, a, b).contains(s) = upper_sum_set(g, a, b).contains(s)
            }
            upper_sum_set(f, a, b).contains(s) or not upper_sum_set(f, a, b).contains(s)
            upper_sum_set(f, a, b).contains(s) = upper_sum_set(g, a, b).contains(s)
        }
        set_ext(upper_sum_set(f, a, b), upper_sum_set(g, a, b))
        upper_sum_set(f, a, b) = upper_sum_set(g, a, b)
    }
}

/// Functions equal on the whole closed interval [a, b] have equal integrals
/// there, provided both are integrable and f is bounded on [a, b].
theorem integral_eq_of_pointwise_eq_on(f: Real -> Real, g: Real -> Real, a: Real, b: Real, lb: Real, ub: Real) {
    a <= b and is_integrable(f, a, b) and is_integrable(g, a, b) and
    (forall(t: Real) { interval_contains(a, b, t) implies f(t) = g(t) }) and
    (forall(t: Real) { interval_contains(a, b, t) implies lb <= f(t) }) and
    (forall(t: Real) { interval_contains(a, b, t) implies f(t) <= ub })
    implies
    integral(f, a, b) = integral(g, a, b)
} by {
    if a <= b and is_integrable(f, a, b) and is_integrable(g, a, b) and
       (forall(t: Real) { interval_contains(a, b, t) implies f(t) = g(t) }) and
       (forall(t: Real) { interval_contains(a, b, t) implies lb <= f(t) }) and
       (forall(t: Real) { interval_contains(a, b, t) implies f(t) <= ub }) {
        integral_spec(f, a, b)
        is_set_supremum(lower_sum_set(f, a, b), integral(f, a, b)) and is_set_infimum(upper_sum_set(f, a, b), integral(f, a, b))
        is_set_supremum(lower_sum_set(f, a, b), integral(f, a, b))
        is_set_infimum(upper_sum_set(f, a, b), integral(f, a, b))
        integral_spec(g, a, b)
        is_set_supremum(lower_sum_set(g, a, b), integral(g, a, b)) and is_set_infimum(upper_sum_set(g, a, b), integral(g, a, b))
        is_set_supremum(lower_sum_set(g, a, b), integral(g, a, b))
        is_set_infimum(upper_sum_set(g, a, b), integral(g, a, b))
        lower_sum_set_eq_of_pointwise_eq_on(f, g, a, b, lb)
        lower_sum_set(f, a, b) = lower_sum_set(g, a, b)
        set_supremum_unique(lower_sum_set(f, a, b), integral(f, a, b), integral(g, a, b))
        integral(f, a, b) = integral(g, a, b)
    }
}

/// If f agrees with an integrable function g on [a, b] and f is bounded on
/// [a, b], then f is integrable on [a, b].
theorem is_integrable_eq_of_pointwise_eq_on(f: Real -> Real, g: Real -> Real, a: Real, b: Real, lb: Real, ub: Real) {
    a <= b and is_integrable(g, a, b) and
    (forall(t: Real) { interval_contains(a, b, t) implies f(t) = g(t) }) and
    (forall(t: Real) { interval_contains(a, b, t) implies lb <= f(t) }) and
    (forall(t: Real) { interval_contains(a, b, t) implies f(t) <= ub })
    implies
    is_integrable(f, a, b)
} by {
    if a <= b and is_integrable(g, a, b) and
       (forall(t: Real) { interval_contains(a, b, t) implies f(t) = g(t) }) and
       (forall(t: Real) { interval_contains(a, b, t) implies lb <= f(t) }) and
       (forall(t: Real) { interval_contains(a, b, t) implies f(t) <= ub }) {
        is_integrable(g, a, b) = exists(m: Real) {
            is_set_supremum(lower_sum_set(g, a, b), m) and is_set_infimum(upper_sum_set(g, a, b), m)
        }
        let m: Real satisfy {
            is_set_supremum(lower_sum_set(g, a, b), m) and is_set_infimum(upper_sum_set(g, a, b), m)
        }
        is_set_supremum(lower_sum_set(g, a, b), m)
        is_set_infimum(upper_sum_set(g, a, b), m)
        lower_sum_set_eq_of_pointwise_eq_on(f, g, a, b, lb)
        lower_sum_set(f, a, b) = lower_sum_set(g, a, b)
        upper_sum_set_eq_of_pointwise_eq_on(f, g, a, b, ub)
        upper_sum_set(f, a, b) = upper_sum_set(g, a, b)
        is_set_supremum(lower_sum_set(f, a, b), m)
        is_set_infimum(upper_sum_set(f, a, b), m)
        is_set_supremum(lower_sum_set(f, a, b), m) and is_set_infimum(upper_sum_set(f, a, b), m)
        exists(m0: Real) {
            is_set_supremum(lower_sum_set(f, a, b), m0) and is_set_infimum(upper_sum_set(f, a, b), m0)
        }
        is_integrable(f, a, b) = exists(m1: Real) {
            is_set_supremum(lower_sum_set(f, a, b), m1) and is_set_infimum(upper_sum_set(f, a, b), m1)
        }
        is_integrable(f, a, b)
    }
}
