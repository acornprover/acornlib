from data.basic.set import Set, compl_contains_eq, double_inclusion, empty_set_contains_eq,
    intersection_contains_eq, set_eq_empty_of_is_empty, subset_contains
from real.real_field import Real
from real.topology import empty_set_is_closed, is_closed_set, is_isolated_point_of_set
from real.topology_derived_set import derived_set, derived_set_member_not_isolated,
    derived_set_subset_of_closed_set, isolated_point_not_in_derived_set
from real.topology_isolated_set import isolated_set, isolated_set_contains_eq,
    isolated_set_member_in_set

/// True if a real set is closed and every point of the set is a limit point of the set.
define is_perfect_real_set(s: Set[Real]) -> Bool {
    is_closed_set(s) and s.subset(derived_set(s))
}

/// A perfect real set is closed.
theorem perfect_real_set_is_closed(s: Set[Real]) {
    is_perfect_real_set(s) implies is_closed_set(s)
} by {
    if is_perfect_real_set(s) {
        is_perfect_real_set(s) = (is_closed_set(s) and s.subset(derived_set(s)))
        is_closed_set(s)
    }
}

/// Every point of a perfect real set is a limit point of that set.
theorem perfect_real_set_subset_derived_set(s: Set[Real]) {
    is_perfect_real_set(s) implies s.subset(derived_set(s))
} by {
    if is_perfect_real_set(s) {
        is_perfect_real_set(s) = (is_closed_set(s) and s.subset(derived_set(s)))
        s.subset(derived_set(s))
    }
}

/// A closed real set contained in its derived set is perfect.
theorem perfect_real_set_intro(s: Set[Real]) {
    is_closed_set(s) and s.subset(derived_set(s)) implies is_perfect_real_set(s)
}

/// The derived set of a perfect real set is contained in the original set.
theorem perfect_real_set_derived_set_subset(s: Set[Real]) {
    is_perfect_real_set(s) implies derived_set(s).subset(s)
} by {
    if is_perfect_real_set(s) {
        perfect_real_set_is_closed(s)
        is_closed_set(s)
        derived_set_subset_of_closed_set(s)
        derived_set(s).subset(s)
    }
}

/// A perfect real set is equal to its derived set.
theorem perfect_real_set_eq_derived_set(s: Set[Real]) {
    is_perfect_real_set(s) implies s = derived_set(s)
} by {
    if is_perfect_real_set(s) {
        perfect_real_set_subset_derived_set(s)
        perfect_real_set_derived_set_subset(s)
        s.subset(derived_set(s))
        derived_set(s).subset(s)
        double_inclusion[Real](s, derived_set(s))
        s = derived_set(s)
    }
}

/// The derived set of a perfect real set is the original set.
theorem derived_set_of_perfect_real_set_eq_self(s: Set[Real]) {
    is_perfect_real_set(s) implies derived_set(s) = s
} by {
    if is_perfect_real_set(s) {
        perfect_real_set_eq_derived_set(s)
        s = derived_set(s)
        derived_set(s) = s
    }
}

/// A point of a perfect real set is not isolated in that set.
theorem perfect_real_set_member_not_isolated(s: Set[Real], x: Real) {
    is_perfect_real_set(s) and s.contains(x) implies not is_isolated_point_of_set(s, x)
} by {
    if is_perfect_real_set(s) and s.contains(x) {
        perfect_real_set_subset_derived_set(s)
        s.subset(derived_set(s))
        subset_contains(s, derived_set(s), x)
        derived_set(s).contains(x)
        derived_set_member_not_isolated(s, x)
        not is_isolated_point_of_set(s, x)
    }
}

/// No point of a perfect real set belongs to its isolated-point set.
theorem perfect_real_set_no_isolated_set_members(s: Set[Real], x: Real) {
    is_perfect_real_set(s) implies not isolated_set(s).contains(x)
} by {
    if is_perfect_real_set(s) {
        if isolated_set(s).contains(x) {
            isolated_set_member_in_set(s, x)
            s.contains(x)
            perfect_real_set_member_not_isolated(s, x)
            not is_isolated_point_of_set(s, x)
            isolated_set_contains_eq(s, x)
            is_isolated_point_of_set(s, x)
            false
        }
    }
}

/// The isolated-point set of a perfect real set is empty.
theorem perfect_real_set_isolated_set_is_empty(s: Set[Real]) {
    is_perfect_real_set(s) implies isolated_set(s).is_empty
} by {
    if is_perfect_real_set(s) {
        forall(x: Real) {
            perfect_real_set_no_isolated_set_members(s, x)
            not isolated_set(s).contains(x)
        }
    }
}

/// The isolated-point set of a perfect real set is the empty set.
theorem perfect_real_set_isolated_set_eq_empty(s: Set[Real]) {
    is_perfect_real_set(s) implies isolated_set(s) = Set[Real].empty_set
} by {
    if is_perfect_real_set(s) {
        perfect_real_set_isolated_set_is_empty(s)
        isolated_set(s).is_empty
        set_eq_empty_of_is_empty[Real](isolated_set(s))
        isolated_set(s) = Set[Real].empty_set
    }
}

/// A member of the isolated-point set is not a member of the derived set.
theorem isolated_set_member_not_in_derived_set(s: Set[Real], x: Real) {
    isolated_set(s).contains(x) implies not derived_set(s).contains(x)
} by {
    if isolated_set(s).contains(x) {
        isolated_set_contains_eq(s, x)
        is_isolated_point_of_set(s, x)
        isolated_point_not_in_derived_set(s, x)
        not derived_set(s).contains(x)
    }
}

/// A member of the derived set is not a member of the isolated-point set.
theorem derived_set_member_not_in_isolated_set(s: Set[Real], x: Real) {
    derived_set(s).contains(x) implies not isolated_set(s).contains(x)
} by {
    if derived_set(s).contains(x) {
        derived_set_member_not_isolated(s, x)
        not is_isolated_point_of_set(s, x)
        if isolated_set(s).contains(x) {
            isolated_set_contains_eq(s, x)
            is_isolated_point_of_set(s, x)
            false
        }
    }
}

/// The isolated-point set is contained in the complement of the derived set.
theorem isolated_set_subset_derived_set_complement(s: Set[Real]) {
    isolated_set(s).subset(derived_set(s).c)
} by {
    forall(x: Real) {
        if isolated_set(s).contains(x) {
            isolated_set_member_not_in_derived_set(s, x)
            not derived_set(s).contains(x)
            compl_contains_eq(derived_set(s), x)
            derived_set(s).c.contains(x)
        }
    }
}

/// The derived set is contained in the complement of the isolated-point set.
theorem derived_set_subset_isolated_set_complement(s: Set[Real]) {
    derived_set(s).subset(isolated_set(s).c)
} by {
    forall(x: Real) {
        if derived_set(s).contains(x) {
            derived_set_member_not_in_isolated_set(s, x)
            not isolated_set(s).contains(x)
            compl_contains_eq(isolated_set(s), x)
            isolated_set(s).c.contains(x)
        }
    }
}

/// The isolated-point set is disjoint from the derived set.
theorem isolated_set_disjoint_derived_set(s: Set[Real]) {
    isolated_set(s).is_disjoint(derived_set(s))
} by {
    forall(x: Real) {
        if isolated_set(s).contains(x) and derived_set(s).contains(x) {
            isolated_set_member_not_in_derived_set(s, x)
            false
        }
        not (isolated_set(s).contains(x) and derived_set(s).contains(x))
    }
}

/// The derived set is disjoint from the isolated-point set.
theorem derived_set_disjoint_isolated_set(s: Set[Real]) {
    derived_set(s).is_disjoint(isolated_set(s))
} by {
    forall(x: Real) {
        if derived_set(s).contains(x) and isolated_set(s).contains(x) {
            derived_set_member_not_in_isolated_set(s, x)
            false
        }
        not (derived_set(s).contains(x) and isolated_set(s).contains(x))
    }
}

/// The intersection of the isolated-point set and the derived set is empty.
theorem isolated_set_intersection_derived_set_eq_empty(s: Set[Real]) {
    isolated_set(s).intersection(derived_set(s)) = Set[Real].empty_set
} by {
    let u = isolated_set(s).intersection(derived_set(s))
    forall(x: Real) {
        if u.contains(x) {
            intersection_contains_eq(isolated_set(s), derived_set(s), x)
            isolated_set(s).contains(x)
            derived_set(s).contains(x)
            isolated_set_member_not_in_derived_set(s, x)
            false
        }
        not u.contains(x)
    }
    u.is_empty
    set_eq_empty_of_is_empty[Real](u)
}

/// The intersection of the derived set and the isolated-point set is empty.
theorem derived_set_intersection_isolated_set_eq_empty(s: Set[Real]) {
    derived_set(s).intersection(isolated_set(s)) = Set[Real].empty_set
} by {
    let u = derived_set(s).intersection(isolated_set(s))
    forall(x: Real) {
        if u.contains(x) {
            intersection_contains_eq(derived_set(s), isolated_set(s), x)
            derived_set(s).contains(x)
            isolated_set(s).contains(x)
            derived_set_member_not_in_isolated_set(s, x)
            false
        }
        not u.contains(x)
    }
    u.is_empty
    set_eq_empty_of_is_empty[Real](u)
}

/// The empty real set is perfect.
theorem empty_real_set_is_perfect {
    is_perfect_real_set(Set[Real].empty_set)
} by {
    empty_set_is_closed
    forall(x: Real) {
        if Set[Real].empty_set.contains(x) {
            empty_set_contains_eq[Real](x)
            false
        }
    }
    Set[Real].empty_set.subset(derived_set(Set[Real].empty_set))
    is_closed_set(Set[Real].empty_set)
    is_perfect_real_set(Set[Real].empty_set)
}

/// A closed real set whose points are all limit points is perfect.
theorem closed_set_with_subset_derived_set_is_perfect(s: Set[Real]) {
    is_closed_set(s) and s.subset(derived_set(s)) implies is_perfect_real_set(s)
} by {
    perfect_real_set_intro(s)
}
