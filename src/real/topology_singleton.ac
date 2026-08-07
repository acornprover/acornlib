from data.basic.set import Set, difference_contains_eq
from real.real_field import Real
from real.real_base import close_comm
from real.real_seq import neq_imp_abs_diff_pos
from real.topology import is_closed_set, is_adherent_point_of_set, is_eps_adherent_to_set,
    is_isolated_point_of_set, is_limit_point_of_set, adherent_point_eps

numerals Real

/// A singleton set of real numbers is closed.
theorem singleton_real_set_is_closed(a: Real) {
    is_closed_set(Set[Real].singleton(a))
} by {
    forall(x: Real) {
        if is_adherent_point_of_set(Set[Real].singleton(a), x) {
            if x != a {
                neq_imp_abs_diff_pos(x, a)
                (x - a).abs.is_positive
                let eps = (x - a).abs
                adherent_point_eps(Set[Real].singleton(a), x, eps)
                is_eps_adherent_to_set(Set[Real].singleton(a), x, eps)
                let y: Real satisfy {
                    Set[Real].singleton(a).contains(y) and y.is_close(x, eps)
                }
                y = a
                a.is_close(x, eps)
                close_comm(a, x, eps)
                x.is_close(a, eps)
                (x - a).abs < eps
                false
            }
            x = a
            Set[Real].singleton(a).contains(x)
        }
    }
}

/// The point of a singleton is isolated in that singleton.
theorem singleton_real_set_isolated(a: Real) {
    is_isolated_point_of_set(Set[Real].singleton(a), a)
} by {
    Set[Real].singleton(a).contains(a)
    Real.1.is_positive
    forall(y: Real) {
        if Set[Real].singleton(a).contains(y) and y != a {
            y = a
            false
        }
    }
    exists(eps: Real) {
        eps.is_positive and forall(y: Real) {
            Set[Real].singleton(a).contains(y) and y != a implies not y.is_close(a, eps)
        }
    }
}

/// A singleton set of real numbers has no limit points.
theorem singleton_real_set_has_no_limit_points(a: Real, x: Real) {
    not is_limit_point_of_set(Set[Real].singleton(a), x)
} by {
    if is_limit_point_of_set(Set[Real].singleton(a), x) {
        let t = Set[Real].singleton(a).difference(Set[Real].singleton(x))
        is_adherent_point_of_set(t, x)
        Real.1.is_positive
        adherent_point_eps(t, x, Real.1)
        is_eps_adherent_to_set(t, x, Real.1)
        let y: Real satisfy {
            t.contains(y) and y.is_close(x, Real.1)
        }
        difference_contains_eq(Set[Real].singleton(a), Set[Real].singleton(x), y)
        Set[Real].singleton(a).contains(y)
        y = a
        not Set[Real].singleton(x).contains(y)
        y != x
        a != x
        neq_imp_abs_diff_pos(a, x)
        let eps2 = (a - x).abs
        eps2.is_positive
        adherent_point_eps(t, x, eps2)
        is_eps_adherent_to_set(t, x, eps2)
        let z: Real satisfy {
            t.contains(z) and z.is_close(x, eps2)
        }
        difference_contains_eq(Set[Real].singleton(a), Set[Real].singleton(x), z)
        Set[Real].singleton(a).contains(z)
        z = a
        a.is_close(x, eps2)
        (a - x).abs < eps2
        false
    }
}
