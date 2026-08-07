from data.basic.function_algebra import pointwise_add, pointwise_mul
from data.basic.functions import identity_fn
from real.derivative_basic import has_derivative_at, differentiable_at
from real.derivative_linear import affine_identity_has_derivative_at,
    affine_identity_differentiable_at
from real.derivative_product import derivative_pointwise_mul,
    derivative_pointwise_square, differentiable_pointwise_mul,
    differentiable_pointwise_square
from real.derivative_repeated_product import derivative_pointwise_cube,
    derivative_pointwise_mul_three_left, differentiable_pointwise_cube,
    differentiable_pointwise_mul_three_left
from real.real_base import Real

/// Multiplying a differentiable function by an affine factor on the left follows the product rule.
theorem derivative_affine_mul(
    c: Real, b: Real, f: Real -> Real, x0: Real, d: Real
) {
    has_derivative_at(f, x0, d)
    implies has_derivative_at(
        pointwise_mul(
            pointwise_add(pointwise_mul(constant[Real, Real](c), identity_fn[Real]), constant[Real, Real](b)),
            f
        ),
        x0,
        pointwise_add(pointwise_mul(constant[Real, Real](c), identity_fn[Real]), constant[Real, Real](b), x0) * d +
        f(x0) * c
    )
} by {
    let affine = pointwise_add(pointwise_mul(constant[Real, Real](c), identity_fn[Real]), constant[Real, Real](b))
    affine_identity_has_derivative_at(c, b, x0)
    has_derivative_at(affine, x0, c)
    derivative_pointwise_mul(affine, f, x0, c, d)
    has_derivative_at(pointwise_mul(affine, f), x0, affine(x0) * d + f(x0) * c)
}

/// Multiplying a differentiable function by an affine factor on the right follows the product rule.
theorem derivative_mul_affine(
    f: Real -> Real, c: Real, b: Real, x0: Real, d: Real
) {
    has_derivative_at(f, x0, d)
    implies has_derivative_at(
        pointwise_mul(
            f,
            pointwise_add(pointwise_mul(constant[Real, Real](c), identity_fn[Real]), constant[Real, Real](b))
        ),
        x0,
        f(x0) * c +
        pointwise_add(pointwise_mul(constant[Real, Real](c), identity_fn[Real]), constant[Real, Real](b), x0) * d
    )
} by {
    let affine = pointwise_add(pointwise_mul(constant[Real, Real](c), identity_fn[Real]), constant[Real, Real](b))
    affine_identity_has_derivative_at(c, b, x0)
    has_derivative_at(affine, x0, c)
    derivative_pointwise_mul(f, affine, x0, d, c)
    has_derivative_at(pointwise_mul(f, affine), x0, f(x0) * c + affine(x0) * d)
}

/// Differentiability is preserved by multiplying on the left by an affine factor.
theorem differentiable_affine_mul(c: Real, b: Real, f: Real -> Real, x0: Real) {
    differentiable_at(f, x0)
    implies differentiable_at(
        pointwise_mul(
            pointwise_add(pointwise_mul(constant[Real, Real](c), identity_fn[Real]), constant[Real, Real](b)),
            f
        ),
        x0
    )
} by {
    let affine = pointwise_add(pointwise_mul(constant[Real, Real](c), identity_fn[Real]), constant[Real, Real](b))
    affine_identity_differentiable_at(c, b, x0)
    differentiable_at(affine, x0)
    differentiable_pointwise_mul(affine, f, x0)
    differentiable_at(pointwise_mul(affine, f), x0)
}

/// Differentiability is preserved by multiplying on the right by an affine factor.
theorem differentiable_mul_affine(f: Real -> Real, c: Real, b: Real, x0: Real) {
    differentiable_at(f, x0)
    implies differentiable_at(
        pointwise_mul(
            f,
            pointwise_add(pointwise_mul(constant[Real, Real](c), identity_fn[Real]), constant[Real, Real](b))
        ),
        x0
    )
} by {
    let affine = pointwise_add(pointwise_mul(constant[Real, Real](c), identity_fn[Real]), constant[Real, Real](b))
    affine_identity_differentiable_at(c, b, x0)
    differentiable_at(affine, x0)
    differentiable_pointwise_mul(f, affine, x0)
    differentiable_at(pointwise_mul(f, affine), x0)
}

/// The product of two affine real functions has derivative from the product rule.
theorem derivative_affine_mul_affine(c: Real, b: Real, a: Real, r: Real, x0: Real) {
    has_derivative_at(
        pointwise_mul(
            pointwise_add(pointwise_mul(constant[Real, Real](c), identity_fn[Real]), constant[Real, Real](b)),
            pointwise_add(pointwise_mul(constant[Real, Real](a), identity_fn[Real]), constant[Real, Real](r))
        ),
        x0,
        pointwise_add(pointwise_mul(constant[Real, Real](c), identity_fn[Real]), constant[Real, Real](b), x0) * a +
        pointwise_add(pointwise_mul(constant[Real, Real](a), identity_fn[Real]), constant[Real, Real](r), x0) * c
    )
} by {
    let affine_c = pointwise_add(pointwise_mul(constant[Real, Real](c), identity_fn[Real]), constant[Real, Real](b))
    let affine_a = pointwise_add(pointwise_mul(constant[Real, Real](a), identity_fn[Real]), constant[Real, Real](r))
    affine_identity_has_derivative_at(c, b, x0)
    has_derivative_at(affine_c, x0, c)
    affine_identity_has_derivative_at(a, r, x0)
    has_derivative_at(affine_a, x0, a)
    derivative_pointwise_mul(affine_c, affine_a, x0, c, a)
    has_derivative_at(pointwise_mul(affine_c, affine_a), x0, affine_c(x0) * a + affine_a(x0) * c)
}

/// The product of two affine real functions is differentiable at every point.
theorem differentiable_affine_mul_affine(c: Real, b: Real, a: Real, r: Real, x0: Real) {
    differentiable_at(
        pointwise_mul(
            pointwise_add(pointwise_mul(constant[Real, Real](c), identity_fn[Real]), constant[Real, Real](b)),
            pointwise_add(pointwise_mul(constant[Real, Real](a), identity_fn[Real]), constant[Real, Real](r))
        ),
        x0
    )
} by {
    let affine_c = pointwise_add(pointwise_mul(constant[Real, Real](c), identity_fn[Real]), constant[Real, Real](b))
    let affine_a = pointwise_add(pointwise_mul(constant[Real, Real](a), identity_fn[Real]), constant[Real, Real](r))
    affine_identity_differentiable_at(c, b, x0)
    differentiable_at(affine_c, x0)
    affine_identity_differentiable_at(a, r, x0)
    differentiable_at(affine_a, x0)
    differentiable_pointwise_mul(affine_c, affine_a, x0)
    differentiable_at(pointwise_mul(affine_c, affine_a), x0)
}

/// The square of an affine real function has derivative from the square rule.
theorem derivative_affine_square(c: Real, b: Real, x0: Real) {
    has_derivative_at(
        pointwise_mul(
            pointwise_add(pointwise_mul(constant[Real, Real](c), identity_fn[Real]), constant[Real, Real](b)),
            pointwise_add(pointwise_mul(constant[Real, Real](c), identity_fn[Real]), constant[Real, Real](b))
        ),
        x0,
        pointwise_add(pointwise_mul(constant[Real, Real](c), identity_fn[Real]), constant[Real, Real](b), x0) * c +
        pointwise_add(pointwise_mul(constant[Real, Real](c), identity_fn[Real]), constant[Real, Real](b), x0) * c
    )
} by {
    let affine = pointwise_add(pointwise_mul(constant[Real, Real](c), identity_fn[Real]), constant[Real, Real](b))
    affine_identity_has_derivative_at(c, b, x0)
    has_derivative_at(affine, x0, c)
    derivative_pointwise_square(affine, x0, c)
    has_derivative_at(pointwise_mul(affine, affine), x0, affine(x0) * c + affine(x0) * c)
}

/// The square of an affine real function is differentiable at every point.
theorem differentiable_affine_square(c: Real, b: Real, x0: Real) {
    differentiable_at(
        pointwise_mul(
            pointwise_add(pointwise_mul(constant[Real, Real](c), identity_fn[Real]), constant[Real, Real](b)),
            pointwise_add(pointwise_mul(constant[Real, Real](c), identity_fn[Real]), constant[Real, Real](b))
        ),
        x0
    )
} by {
    let affine = pointwise_add(pointwise_mul(constant[Real, Real](c), identity_fn[Real]), constant[Real, Real](b))
    affine_identity_differentiable_at(c, b, x0)
    differentiable_at(affine, x0)
    differentiable_pointwise_square(affine, x0)
    differentiable_at(pointwise_mul(affine, affine), x0)
}

/// The cube of an affine real function has derivative from the triple product rule.
theorem derivative_affine_cube(c: Real, b: Real, x0: Real) {
    has_derivative_at(
        pointwise_mul(
            pointwise_mul(
                pointwise_add(pointwise_mul(constant[Real, Real](c), identity_fn[Real]), constant[Real, Real](b)),
                pointwise_add(pointwise_mul(constant[Real, Real](c), identity_fn[Real]), constant[Real, Real](b))
            ),
            pointwise_add(pointwise_mul(constant[Real, Real](c), identity_fn[Real]), constant[Real, Real](b))
        ),
        x0,
        pointwise_mul(
            pointwise_add(pointwise_mul(constant[Real, Real](c), identity_fn[Real]), constant[Real, Real](b)),
            pointwise_add(pointwise_mul(constant[Real, Real](c), identity_fn[Real]), constant[Real, Real](b)),
            x0
        ) * c +
        pointwise_add(pointwise_mul(constant[Real, Real](c), identity_fn[Real]), constant[Real, Real](b), x0) *
        (
            pointwise_add(pointwise_mul(constant[Real, Real](c), identity_fn[Real]), constant[Real, Real](b), x0) * c +
            pointwise_add(pointwise_mul(constant[Real, Real](c), identity_fn[Real]), constant[Real, Real](b), x0) * c
        )
    )
} by {
    let affine = pointwise_add(pointwise_mul(constant[Real, Real](c), identity_fn[Real]), constant[Real, Real](b))
    affine_identity_has_derivative_at(c, b, x0)
    has_derivative_at(affine, x0, c)
    derivative_pointwise_cube(affine, x0, c)
    has_derivative_at(
        pointwise_mul(pointwise_mul(affine, affine), affine),
        x0,
        pointwise_mul(affine, affine, x0) * c + affine(x0) * (affine(x0) * c + affine(x0) * c)
    )
}

/// The cube of an affine real function is differentiable at every point.
theorem differentiable_affine_cube(c: Real, b: Real, x0: Real) {
    differentiable_at(
        pointwise_mul(
            pointwise_mul(
                pointwise_add(pointwise_mul(constant[Real, Real](c), identity_fn[Real]), constant[Real, Real](b)),
                pointwise_add(pointwise_mul(constant[Real, Real](c), identity_fn[Real]), constant[Real, Real](b))
            ),
            pointwise_add(pointwise_mul(constant[Real, Real](c), identity_fn[Real]), constant[Real, Real](b))
        ),
        x0
    )
} by {
    let affine = pointwise_add(pointwise_mul(constant[Real, Real](c), identity_fn[Real]), constant[Real, Real](b))
    affine_identity_differentiable_at(c, b, x0)
    differentiable_at(affine, x0)
    differentiable_pointwise_cube(affine, x0)
    differentiable_at(pointwise_mul(pointwise_mul(affine, affine), affine), x0)
}

/// Multiplying a differentiable function by an affine square on the left follows the product rule.
theorem derivative_affine_square_mul(
    c: Real, b: Real, f: Real -> Real, x0: Real, d: Real
) {
    has_derivative_at(f, x0, d)
    implies has_derivative_at(
        pointwise_mul(
            pointwise_mul(
                pointwise_add(pointwise_mul(constant[Real, Real](c), identity_fn[Real]), constant[Real, Real](b)),
                pointwise_add(pointwise_mul(constant[Real, Real](c), identity_fn[Real]), constant[Real, Real](b))
            ),
            f
        ),
        x0,
        pointwise_mul(
            pointwise_add(pointwise_mul(constant[Real, Real](c), identity_fn[Real]), constant[Real, Real](b)),
            pointwise_add(pointwise_mul(constant[Real, Real](c), identity_fn[Real]), constant[Real, Real](b)),
            x0
        ) * d +
        f(x0) * (
            pointwise_add(pointwise_mul(constant[Real, Real](c), identity_fn[Real]), constant[Real, Real](b), x0) * c +
            pointwise_add(pointwise_mul(constant[Real, Real](c), identity_fn[Real]), constant[Real, Real](b), x0) * c
        )
    )
} by {
    let affine = pointwise_add(pointwise_mul(constant[Real, Real](c), identity_fn[Real]), constant[Real, Real](b))
    derivative_affine_square(c, b, x0)
    has_derivative_at(pointwise_mul(affine, affine), x0, affine(x0) * c + affine(x0) * c)
    derivative_pointwise_mul(pointwise_mul(affine, affine), f, x0, affine(x0) * c + affine(x0) * c, d)
    has_derivative_at(
        pointwise_mul(pointwise_mul(affine, affine), f),
        x0,
        pointwise_mul(affine, affine, x0) * d + f(x0) * (affine(x0) * c + affine(x0) * c)
    )
}

/// Multiplying a differentiable function by an affine square on the right follows the product rule.
theorem derivative_mul_affine_square(
    f: Real -> Real, c: Real, b: Real, x0: Real, d: Real
) {
    has_derivative_at(f, x0, d)
    implies has_derivative_at(
        pointwise_mul(
            f,
            pointwise_mul(
                pointwise_add(pointwise_mul(constant[Real, Real](c), identity_fn[Real]), constant[Real, Real](b)),
                pointwise_add(pointwise_mul(constant[Real, Real](c), identity_fn[Real]), constant[Real, Real](b))
            )
        ),
        x0,
        f(x0) * (
            pointwise_add(pointwise_mul(constant[Real, Real](c), identity_fn[Real]), constant[Real, Real](b), x0) * c +
            pointwise_add(pointwise_mul(constant[Real, Real](c), identity_fn[Real]), constant[Real, Real](b), x0) * c
        ) +
        pointwise_mul(
            pointwise_add(pointwise_mul(constant[Real, Real](c), identity_fn[Real]), constant[Real, Real](b)),
            pointwise_add(pointwise_mul(constant[Real, Real](c), identity_fn[Real]), constant[Real, Real](b)),
            x0
        ) * d
    )
} by {
    let affine = pointwise_add(pointwise_mul(constant[Real, Real](c), identity_fn[Real]), constant[Real, Real](b))
    derivative_affine_square(c, b, x0)
    has_derivative_at(pointwise_mul(affine, affine), x0, affine(x0) * c + affine(x0) * c)
    derivative_pointwise_mul(f, pointwise_mul(affine, affine), x0, d, affine(x0) * c + affine(x0) * c)
    has_derivative_at(
        pointwise_mul(f, pointwise_mul(affine, affine)),
        x0,
        f(x0) * (affine(x0) * c + affine(x0) * c) + pointwise_mul(affine, affine, x0) * d
    )
}

/// Differentiability is preserved by multiplying on the left by an affine square.
theorem differentiable_affine_square_mul(c: Real, b: Real, f: Real -> Real, x0: Real) {
    differentiable_at(f, x0)
    implies differentiable_at(
        pointwise_mul(
            pointwise_mul(
                pointwise_add(pointwise_mul(constant[Real, Real](c), identity_fn[Real]), constant[Real, Real](b)),
                pointwise_add(pointwise_mul(constant[Real, Real](c), identity_fn[Real]), constant[Real, Real](b))
            ),
            f
        ),
        x0
    )
} by {
    let affine = pointwise_add(pointwise_mul(constant[Real, Real](c), identity_fn[Real]), constant[Real, Real](b))
    differentiable_affine_square(c, b, x0)
    differentiable_at(pointwise_mul(affine, affine), x0)
    differentiable_pointwise_mul(pointwise_mul(affine, affine), f, x0)
    differentiable_at(pointwise_mul(pointwise_mul(affine, affine), f), x0)
}

/// Differentiability is preserved by multiplying on the right by an affine square.
theorem differentiable_mul_affine_square(f: Real -> Real, c: Real, b: Real, x0: Real) {
    differentiable_at(f, x0)
    implies differentiable_at(
        pointwise_mul(
            f,
            pointwise_mul(
                pointwise_add(pointwise_mul(constant[Real, Real](c), identity_fn[Real]), constant[Real, Real](b)),
                pointwise_add(pointwise_mul(constant[Real, Real](c), identity_fn[Real]), constant[Real, Real](b))
            )
        ),
        x0
    )
} by {
    let affine = pointwise_add(pointwise_mul(constant[Real, Real](c), identity_fn[Real]), constant[Real, Real](b))
    differentiable_affine_square(c, b, x0)
    differentiable_at(pointwise_mul(affine, affine), x0)
    differentiable_pointwise_mul(f, pointwise_mul(affine, affine), x0)
    differentiable_at(pointwise_mul(f, pointwise_mul(affine, affine)), x0)
}

/// Multiplying a differentiable function by an affine cube on the left follows the product rule.
theorem derivative_affine_cube_mul(
    c: Real, b: Real, f: Real -> Real, x0: Real, d: Real
) {
    has_derivative_at(f, x0, d)
    implies has_derivative_at(
        pointwise_mul(
            pointwise_mul(
                pointwise_mul(
                    pointwise_add(pointwise_mul(constant[Real, Real](c), identity_fn[Real]), constant[Real, Real](b)),
                    pointwise_add(pointwise_mul(constant[Real, Real](c), identity_fn[Real]), constant[Real, Real](b))
                ),
                pointwise_add(pointwise_mul(constant[Real, Real](c), identity_fn[Real]), constant[Real, Real](b))
            ),
            f
        ),
        x0,
        pointwise_mul(
            pointwise_mul(
                pointwise_add(pointwise_mul(constant[Real, Real](c), identity_fn[Real]), constant[Real, Real](b)),
                pointwise_add(pointwise_mul(constant[Real, Real](c), identity_fn[Real]), constant[Real, Real](b))
            ),
            pointwise_add(pointwise_mul(constant[Real, Real](c), identity_fn[Real]), constant[Real, Real](b)),
            x0
        ) * d +
        f(x0) * (
            pointwise_mul(
                pointwise_add(pointwise_mul(constant[Real, Real](c), identity_fn[Real]), constant[Real, Real](b)),
                pointwise_add(pointwise_mul(constant[Real, Real](c), identity_fn[Real]), constant[Real, Real](b)),
                x0
            ) * c +
            pointwise_add(pointwise_mul(constant[Real, Real](c), identity_fn[Real]), constant[Real, Real](b), x0) *
            (
                pointwise_add(pointwise_mul(constant[Real, Real](c), identity_fn[Real]), constant[Real, Real](b), x0) * c +
                pointwise_add(pointwise_mul(constant[Real, Real](c), identity_fn[Real]), constant[Real, Real](b), x0) * c
            )
        )
    )
} by {
    derivative_affine_cube(c, b, x0)
    derivative_pointwise_mul(
        pointwise_mul(
            pointwise_mul(
                pointwise_add(pointwise_mul(constant[Real, Real](c), identity_fn[Real]), constant[Real, Real](b)),
                pointwise_add(pointwise_mul(constant[Real, Real](c), identity_fn[Real]), constant[Real, Real](b))
            ),
            pointwise_add(pointwise_mul(constant[Real, Real](c), identity_fn[Real]), constant[Real, Real](b))
        ),
        f,
        x0,
        pointwise_mul(
            pointwise_add(pointwise_mul(constant[Real, Real](c), identity_fn[Real]), constant[Real, Real](b)),
            pointwise_add(pointwise_mul(constant[Real, Real](c), identity_fn[Real]), constant[Real, Real](b)),
            x0
        ) * c +
        pointwise_add(pointwise_mul(constant[Real, Real](c), identity_fn[Real]), constant[Real, Real](b), x0) *
        (
            pointwise_add(pointwise_mul(constant[Real, Real](c), identity_fn[Real]), constant[Real, Real](b), x0) * c +
            pointwise_add(pointwise_mul(constant[Real, Real](c), identity_fn[Real]), constant[Real, Real](b), x0) * c
        ),
        d
    )
}

/// Multiplying a differentiable function by an affine cube on the right follows the product rule.
theorem derivative_mul_affine_cube(
    f: Real -> Real, c: Real, b: Real, x0: Real, d: Real
) {
    has_derivative_at(f, x0, d)
    implies has_derivative_at(
        pointwise_mul(
            f,
            pointwise_mul(
                pointwise_mul(
                    pointwise_add(pointwise_mul(constant[Real, Real](c), identity_fn[Real]), constant[Real, Real](b)),
                    pointwise_add(pointwise_mul(constant[Real, Real](c), identity_fn[Real]), constant[Real, Real](b))
                ),
                pointwise_add(pointwise_mul(constant[Real, Real](c), identity_fn[Real]), constant[Real, Real](b))
            )
        ),
        x0,
        f(x0) * (
            pointwise_mul(
                pointwise_add(pointwise_mul(constant[Real, Real](c), identity_fn[Real]), constant[Real, Real](b)),
                pointwise_add(pointwise_mul(constant[Real, Real](c), identity_fn[Real]), constant[Real, Real](b)),
                x0
            ) * c +
            pointwise_add(pointwise_mul(constant[Real, Real](c), identity_fn[Real]), constant[Real, Real](b), x0) *
            (
                pointwise_add(pointwise_mul(constant[Real, Real](c), identity_fn[Real]), constant[Real, Real](b), x0) * c +
                pointwise_add(pointwise_mul(constant[Real, Real](c), identity_fn[Real]), constant[Real, Real](b), x0) * c
            )
        ) +
        pointwise_mul(
            pointwise_mul(
                pointwise_add(pointwise_mul(constant[Real, Real](c), identity_fn[Real]), constant[Real, Real](b)),
                pointwise_add(pointwise_mul(constant[Real, Real](c), identity_fn[Real]), constant[Real, Real](b))
            ),
            pointwise_add(pointwise_mul(constant[Real, Real](c), identity_fn[Real]), constant[Real, Real](b)),
            x0
        ) * d
    )
} by {
    derivative_affine_cube(c, b, x0)
    derivative_pointwise_mul(
        f,
        pointwise_mul(
            pointwise_mul(
                pointwise_add(pointwise_mul(constant[Real, Real](c), identity_fn[Real]), constant[Real, Real](b)),
                pointwise_add(pointwise_mul(constant[Real, Real](c), identity_fn[Real]), constant[Real, Real](b))
            ),
            pointwise_add(pointwise_mul(constant[Real, Real](c), identity_fn[Real]), constant[Real, Real](b))
        ),
        x0,
        d,
        pointwise_mul(
            pointwise_add(pointwise_mul(constant[Real, Real](c), identity_fn[Real]), constant[Real, Real](b)),
            pointwise_add(pointwise_mul(constant[Real, Real](c), identity_fn[Real]), constant[Real, Real](b)),
            x0
        ) * c +
        pointwise_add(pointwise_mul(constant[Real, Real](c), identity_fn[Real]), constant[Real, Real](b), x0) *
        (
            pointwise_add(pointwise_mul(constant[Real, Real](c), identity_fn[Real]), constant[Real, Real](b), x0) * c +
            pointwise_add(pointwise_mul(constant[Real, Real](c), identity_fn[Real]), constant[Real, Real](b), x0) * c
        )
    )
}

/// Differentiability is preserved by multiplying on the left by an affine cube.
theorem differentiable_affine_cube_mul(c: Real, b: Real, f: Real -> Real, x0: Real) {
    differentiable_at(f, x0)
    implies differentiable_at(
        pointwise_mul(
            pointwise_mul(
                pointwise_mul(
                    pointwise_add(pointwise_mul(constant[Real, Real](c), identity_fn[Real]), constant[Real, Real](b)),
                    pointwise_add(pointwise_mul(constant[Real, Real](c), identity_fn[Real]), constant[Real, Real](b))
                ),
                pointwise_add(pointwise_mul(constant[Real, Real](c), identity_fn[Real]), constant[Real, Real](b))
            ),
            f
        ),
        x0
    )
} by {
    let affine = pointwise_add(pointwise_mul(constant[Real, Real](c), identity_fn[Real]), constant[Real, Real](b))
    differentiable_affine_cube(c, b, x0)
    differentiable_at(pointwise_mul(pointwise_mul(affine, affine), affine), x0)
    differentiable_pointwise_mul(pointwise_mul(pointwise_mul(affine, affine), affine), f, x0)
    differentiable_at(pointwise_mul(pointwise_mul(pointwise_mul(affine, affine), affine), f), x0)
}

/// Differentiability is preserved by multiplying on the right by an affine cube.
theorem differentiable_mul_affine_cube(f: Real -> Real, c: Real, b: Real, x0: Real) {
    differentiable_at(f, x0)
    implies differentiable_at(
        pointwise_mul(
            f,
            pointwise_mul(
                pointwise_mul(
                    pointwise_add(pointwise_mul(constant[Real, Real](c), identity_fn[Real]), constant[Real, Real](b)),
                    pointwise_add(pointwise_mul(constant[Real, Real](c), identity_fn[Real]), constant[Real, Real](b))
                ),
                pointwise_add(pointwise_mul(constant[Real, Real](c), identity_fn[Real]), constant[Real, Real](b))
            )
        ),
        x0
    )
} by {
    let affine = pointwise_add(pointwise_mul(constant[Real, Real](c), identity_fn[Real]), constant[Real, Real](b))
    differentiable_affine_cube(c, b, x0)
    differentiable_at(pointwise_mul(pointwise_mul(affine, affine), affine), x0)
    differentiable_pointwise_mul(f, pointwise_mul(pointwise_mul(affine, affine), affine), x0)
    differentiable_at(pointwise_mul(f, pointwise_mul(pointwise_mul(affine, affine), affine)), x0)
}

/// A left-associated product of three affine real functions follows the iterated product rule.
theorem derivative_affine_mul_affine_mul_affine(
    e: Real, s: Real, c: Real, b: Real, a: Real, r: Real, x0: Real
) {
    has_derivative_at(
        pointwise_mul(
            pointwise_mul(
                pointwise_add(pointwise_mul(constant[Real, Real](e), identity_fn[Real]), constant[Real, Real](s)),
                pointwise_add(pointwise_mul(constant[Real, Real](c), identity_fn[Real]), constant[Real, Real](b))
            ),
            pointwise_add(pointwise_mul(constant[Real, Real](a), identity_fn[Real]), constant[Real, Real](r))
        ),
        x0,
        pointwise_mul(
            pointwise_add(pointwise_mul(constant[Real, Real](e), identity_fn[Real]), constant[Real, Real](s)),
            pointwise_add(pointwise_mul(constant[Real, Real](c), identity_fn[Real]), constant[Real, Real](b)),
            x0
        ) * a +
        pointwise_add(pointwise_mul(constant[Real, Real](a), identity_fn[Real]), constant[Real, Real](r), x0) *
        (
            pointwise_add(pointwise_mul(constant[Real, Real](e), identity_fn[Real]), constant[Real, Real](s), x0) * c +
            pointwise_add(pointwise_mul(constant[Real, Real](c), identity_fn[Real]), constant[Real, Real](b), x0) * e
        )
    )
} by {
    let affine_e = pointwise_add(pointwise_mul(constant[Real, Real](e), identity_fn[Real]), constant[Real, Real](s))
    let affine_c = pointwise_add(pointwise_mul(constant[Real, Real](c), identity_fn[Real]), constant[Real, Real](b))
    let affine_a = pointwise_add(pointwise_mul(constant[Real, Real](a), identity_fn[Real]), constant[Real, Real](r))
    affine_identity_has_derivative_at(e, s, x0)
    has_derivative_at(affine_e, x0, e)
    affine_identity_has_derivative_at(c, b, x0)
    has_derivative_at(affine_c, x0, c)
    affine_identity_has_derivative_at(a, r, x0)
    has_derivative_at(affine_a, x0, a)
    derivative_pointwise_mul_three_left(affine_e, affine_c, affine_a, x0, e, c, a)
    has_derivative_at(
        pointwise_mul(pointwise_mul(affine_e, affine_c), affine_a),
        x0,
        pointwise_mul(affine_e, affine_c, x0) * a + affine_a(x0) * (affine_e(x0) * c + affine_c(x0) * e)
    )
}

/// A left-associated product of three affine real functions is differentiable at every point.
theorem differentiable_affine_mul_affine_mul_affine(
    e: Real, s: Real, c: Real, b: Real, a: Real, r: Real, x0: Real
) {
    differentiable_at(
        pointwise_mul(
            pointwise_mul(
                pointwise_add(pointwise_mul(constant[Real, Real](e), identity_fn[Real]), constant[Real, Real](s)),
                pointwise_add(pointwise_mul(constant[Real, Real](c), identity_fn[Real]), constant[Real, Real](b))
            ),
            pointwise_add(pointwise_mul(constant[Real, Real](a), identity_fn[Real]), constant[Real, Real](r))
        ),
        x0
    )
} by {
    let affine_e = pointwise_add(pointwise_mul(constant[Real, Real](e), identity_fn[Real]), constant[Real, Real](s))
    let affine_c = pointwise_add(pointwise_mul(constant[Real, Real](c), identity_fn[Real]), constant[Real, Real](b))
    let affine_a = pointwise_add(pointwise_mul(constant[Real, Real](a), identity_fn[Real]), constant[Real, Real](r))
    affine_identity_differentiable_at(e, s, x0)
    differentiable_at(affine_e, x0)
    affine_identity_differentiable_at(c, b, x0)
    differentiable_at(affine_c, x0)
    affine_identity_differentiable_at(a, r, x0)
    differentiable_at(affine_a, x0)
    differentiable_pointwise_mul_three_left(affine_e, affine_c, affine_a, x0)
    differentiable_at(pointwise_mul(pointwise_mul(affine_e, affine_c), affine_a), x0)
}
