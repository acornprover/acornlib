from order import lte_trans, lt_of_lte_of_lt, lt_of_lt_of_lte
from order_set import closed_interval_set, left_open_interval_set, open_interval_set,
    right_open_interval_set
from real.real_field import Real
from real.topology import disconnecting_triple, is_connected_real_set
from real.topology_connected_algebra import connected_real_set_of_eq_rev,
    intersection_of_connected_real_sets_is_connected
from real.topology_intervals import closed_interval_set_eq_closed_ray_intersection,
    left_open_interval_set_eq_ray_intersection, open_interval_set_eq_open_ray_intersection,
    right_open_interval_set_eq_ray_intersection
from real.topology_rays import closed_lower_ray, closed_lower_ray_contains_eq,
    closed_upper_ray, closed_upper_ray_contains_eq, open_lower_ray, open_lower_ray_contains_eq,
    open_upper_ray, open_upper_ray_contains_eq

/// Open upper rays are connected real sets.
theorem open_upper_ray_is_connected(lower: Real) {
    is_connected_real_set(open_upper_ray(lower))
} by {
    forall(x: Real, y: Real, z: Real) {
        if disconnecting_triple(open_upper_ray(lower), x, y, z) {
            disconnecting_triple(open_upper_ray(lower), x, y, z) =
                (open_upper_ray(lower).contains(x) and open_upper_ray(lower).contains(y) and
                 x < y and x <= z and z <= y and not open_upper_ray(lower).contains(z))
            open_upper_ray(lower).contains(x)
            x < y
            x <= z
            open_upper_ray_contains_eq(lower, x)
            lower < x
            lt_of_lt_of_lte[Real](lower, x, z)
            lower < z
            open_upper_ray_contains_eq(lower, z)
            open_upper_ray(lower).contains(z)
            not open_upper_ray(lower).contains(z)
            false
        }
    }
}

/// Open lower rays are connected real sets.
theorem open_lower_ray_is_connected(upper: Real) {
    is_connected_real_set(open_lower_ray(upper))
} by {
    forall(x: Real, y: Real, z: Real) {
        if disconnecting_triple(open_lower_ray(upper), x, y, z) {
            disconnecting_triple(open_lower_ray(upper), x, y, z) =
                (open_lower_ray(upper).contains(x) and open_lower_ray(upper).contains(y) and
                 x < y and x <= z and z <= y and not open_lower_ray(upper).contains(z))
            open_lower_ray(upper).contains(y)
            z <= y
            open_lower_ray_contains_eq(upper, y)
            y < upper
            lt_of_lte_of_lt[Real](z, y, upper)
            z < upper
            open_lower_ray_contains_eq(upper, z)
            open_lower_ray(upper).contains(z)
            not open_lower_ray(upper).contains(z)
            false
        }
    }
}

/// Closed upper rays are connected real sets.
theorem closed_upper_ray_is_connected(lower: Real) {
    is_connected_real_set(closed_upper_ray(lower))
} by {
    forall(x: Real, y: Real, z: Real) {
        if disconnecting_triple(closed_upper_ray(lower), x, y, z) {
            disconnecting_triple(closed_upper_ray(lower), x, y, z) =
                (closed_upper_ray(lower).contains(x) and closed_upper_ray(lower).contains(y) and
                 x < y and x <= z and z <= y and not closed_upper_ray(lower).contains(z))
            closed_upper_ray(lower).contains(x)
            x <= z
            closed_upper_ray_contains_eq(lower, x)
            lower <= x
            lte_trans[Real](lower, x, z)
            lower <= z
            closed_upper_ray_contains_eq(lower, z)
            closed_upper_ray(lower).contains(z)
            not closed_upper_ray(lower).contains(z)
            false
        }
    }
}

/// Closed lower rays are connected real sets.
theorem closed_lower_ray_is_connected(upper: Real) {
    is_connected_real_set(closed_lower_ray(upper))
} by {
    forall(x: Real, y: Real, z: Real) {
        if disconnecting_triple(closed_lower_ray(upper), x, y, z) {
            disconnecting_triple(closed_lower_ray(upper), x, y, z) =
                (closed_lower_ray(upper).contains(x) and closed_lower_ray(upper).contains(y) and
                 x < y and x <= z and z <= y and not closed_lower_ray(upper).contains(z))
            closed_lower_ray(upper).contains(y)
            z <= y
            closed_lower_ray_contains_eq(upper, y)
            y <= upper
            lte_trans[Real](z, y, upper)
            z <= upper
            closed_lower_ray_contains_eq(upper, z)
            closed_lower_ray(upper).contains(z)
            not closed_lower_ray(upper).contains(z)
            false
        }
    }
}

/// Open intervals are connected real sets.
theorem open_interval_set_is_connected(lower: Real, upper: Real) {
    is_connected_real_set(open_interval_set(lower, upper))
} by {
    open_upper_ray_is_connected(lower)
    open_lower_ray_is_connected(upper)
    intersection_of_connected_real_sets_is_connected(open_upper_ray(lower), open_lower_ray(upper))
    is_connected_real_set(open_upper_ray(lower).intersection(open_lower_ray(upper)))
    open_interval_set_eq_open_ray_intersection(lower, upper)
    open_interval_set(lower, upper) = open_upper_ray(lower).intersection(open_lower_ray(upper))
    connected_real_set_of_eq_rev(open_interval_set(lower, upper),
        open_upper_ray(lower).intersection(open_lower_ray(upper)))
    is_connected_real_set(open_interval_set(lower, upper))
}

/// Closed intervals are connected real sets.
theorem closed_interval_set_is_connected(lower: Real, upper: Real) {
    is_connected_real_set(closed_interval_set(lower, upper))
} by {
    closed_upper_ray_is_connected(lower)
    closed_lower_ray_is_connected(upper)
    intersection_of_connected_real_sets_is_connected(closed_upper_ray(lower), closed_lower_ray(upper))
    is_connected_real_set(closed_upper_ray(lower).intersection(closed_lower_ray(upper)))
    closed_interval_set_eq_closed_ray_intersection(lower, upper)
    closed_interval_set(lower, upper) = closed_upper_ray(lower).intersection(closed_lower_ray(upper))
    connected_real_set_of_eq_rev(closed_interval_set(lower, upper),
        closed_upper_ray(lower).intersection(closed_lower_ray(upper)))
    is_connected_real_set(closed_interval_set(lower, upper))
}

/// Left-open intervals are connected real sets.
theorem left_open_interval_set_is_connected(lower: Real, upper: Real) {
    is_connected_real_set(left_open_interval_set(lower, upper))
} by {
    open_upper_ray_is_connected(lower)
    closed_lower_ray_is_connected(upper)
    intersection_of_connected_real_sets_is_connected(open_upper_ray(lower), closed_lower_ray(upper))
    is_connected_real_set(open_upper_ray(lower).intersection(closed_lower_ray(upper)))
    left_open_interval_set_eq_ray_intersection(lower, upper)
    left_open_interval_set(lower, upper) = open_upper_ray(lower).intersection(closed_lower_ray(upper))
    connected_real_set_of_eq_rev(left_open_interval_set(lower, upper),
        open_upper_ray(lower).intersection(closed_lower_ray(upper)))
    is_connected_real_set(left_open_interval_set(lower, upper))
}

/// Right-open intervals are connected real sets.
theorem right_open_interval_set_is_connected(lower: Real, upper: Real) {
    is_connected_real_set(right_open_interval_set(lower, upper))
} by {
    closed_upper_ray_is_connected(lower)
    open_lower_ray_is_connected(upper)
    intersection_of_connected_real_sets_is_connected(closed_upper_ray(lower), open_lower_ray(upper))
    is_connected_real_set(closed_upper_ray(lower).intersection(open_lower_ray(upper)))
    right_open_interval_set_eq_ray_intersection(lower, upper)
    right_open_interval_set(lower, upper) = closed_upper_ray(lower).intersection(open_lower_ray(upper))
    connected_real_set_of_eq_rev(right_open_interval_set(lower, upper),
        closed_upper_ray(lower).intersection(open_lower_ray(upper)))
    is_connected_real_set(right_open_interval_set(lower, upper))
}
