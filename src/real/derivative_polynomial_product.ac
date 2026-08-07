from data.basic.function_algebra import pointwise_mul
from real.continuity_cube import cube_real
from real.continuity_square import square_real
from real.derivative_basic import has_derivative_at, differentiable_at
from real.derivative_polynomial_chain import cube_real_differentiable_at,
    cube_real_has_derivative_at, square_real_differentiable_at,
    square_real_has_derivative_at
from real.derivative_product import derivative_pointwise_mul,
    differentiable_pointwise_mul
from real.real_base import Real

/// Multiplying a differentiable function by the square function on the left follows the product rule.
theorem derivative_square_real_mul(f: Real -> Real, x0: Real, d: Real) {
    has_derivative_at(f, x0, d)
    implies has_derivative_at(
        pointwise_mul(square_real, f),
        x0,
        square_real(x0) * d + f(x0) * (x0 * Real.1 + x0 * Real.1)
    )
} by {
    square_real_has_derivative_at(x0)
    derivative_pointwise_mul(square_real, f, x0, x0 * Real.1 + x0 * Real.1, d)
    has_derivative_at(pointwise_mul(square_real, f), x0,
        square_real(x0) * d + f(x0) * (x0 * Real.1 + x0 * Real.1))
}

/// Multiplying a differentiable function by the square function on the right follows the product rule.
theorem derivative_mul_square_real(f: Real -> Real, x0: Real, d: Real) {
    has_derivative_at(f, x0, d)
    implies has_derivative_at(
        pointwise_mul(f, square_real),
        x0,
        f(x0) * (x0 * Real.1 + x0 * Real.1) + square_real(x0) * d
    )
} by {
    square_real_has_derivative_at(x0)
    derivative_pointwise_mul(f, square_real, x0, d, x0 * Real.1 + x0 * Real.1)
    has_derivative_at(pointwise_mul(f, square_real), x0,
        f(x0) * (x0 * Real.1 + x0 * Real.1) + square_real(x0) * d)
}

/// Multiplying a differentiable function by the cube function on the left follows the product rule.
theorem derivative_cube_real_mul(f: Real -> Real, x0: Real, d: Real) {
    has_derivative_at(f, x0, d)
    implies has_derivative_at(
        pointwise_mul(cube_real, f),
        x0,
        cube_real(x0) * d + f(x0) * (square_real(x0) * Real.1 + x0 * (x0 * Real.1 + x0 * Real.1))
    )
} by {
    cube_real_has_derivative_at(x0)
    derivative_pointwise_mul(cube_real, f, x0,
        square_real(x0) * Real.1 + x0 * (x0 * Real.1 + x0 * Real.1),
        d)
    has_derivative_at(pointwise_mul(cube_real, f), x0,
        cube_real(x0) * d + f(x0) * (square_real(x0) * Real.1 + x0 * (x0 * Real.1 + x0 * Real.1)))
}

/// Multiplying a differentiable function by the cube function on the right follows the product rule.
theorem derivative_mul_cube_real(f: Real -> Real, x0: Real, d: Real) {
    has_derivative_at(f, x0, d)
    implies has_derivative_at(
        pointwise_mul(f, cube_real),
        x0,
        f(x0) * (square_real(x0) * Real.1 + x0 * (x0 * Real.1 + x0 * Real.1)) + cube_real(x0) * d
    )
} by {
    cube_real_has_derivative_at(x0)
    derivative_pointwise_mul(f, cube_real, x0,
        d,
        square_real(x0) * Real.1 + x0 * (x0 * Real.1 + x0 * Real.1))
    has_derivative_at(pointwise_mul(f, cube_real), x0,
        f(x0) * (square_real(x0) * Real.1 + x0 * (x0 * Real.1 + x0 * Real.1)) + cube_real(x0) * d)
}

/// Differentiability is preserved by multiplying on the left by the square function.
theorem differentiable_square_real_mul(f: Real -> Real, x0: Real) {
    differentiable_at(f, x0) implies differentiable_at(pointwise_mul(square_real, f), x0)
} by {
    square_real_differentiable_at(x0)
    differentiable_pointwise_mul(square_real, f, x0)
    differentiable_at(pointwise_mul(square_real, f), x0)
}

/// Differentiability is preserved by multiplying on the right by the square function.
theorem differentiable_mul_square_real(f: Real -> Real, x0: Real) {
    differentiable_at(f, x0) implies differentiable_at(pointwise_mul(f, square_real), x0)
} by {
    square_real_differentiable_at(x0)
    differentiable_pointwise_mul(f, square_real, x0)
    differentiable_at(pointwise_mul(f, square_real), x0)
}

/// Differentiability is preserved by multiplying on the left by the cube function.
theorem differentiable_cube_real_mul(f: Real -> Real, x0: Real) {
    differentiable_at(f, x0) implies differentiable_at(pointwise_mul(cube_real, f), x0)
} by {
    cube_real_differentiable_at(x0)
    differentiable_pointwise_mul(cube_real, f, x0)
    differentiable_at(pointwise_mul(cube_real, f), x0)
}

/// Differentiability is preserved by multiplying on the right by the cube function.
theorem differentiable_mul_cube_real(f: Real -> Real, x0: Real) {
    differentiable_at(f, x0) implies differentiable_at(pointwise_mul(f, cube_real), x0)
} by {
    cube_real_differentiable_at(x0)
    differentiable_pointwise_mul(f, cube_real, x0)
    differentiable_at(pointwise_mul(f, cube_real), x0)
}

/// The product of the square and cube functions has derivative from the product rule.
theorem derivative_square_real_mul_cube_real(x0: Real) {
    has_derivative_at(
        pointwise_mul(square_real, cube_real),
        x0,
        square_real(x0) * (square_real(x0) * Real.1 + x0 * (x0 * Real.1 + x0 * Real.1)) +
        cube_real(x0) * (x0 * Real.1 + x0 * Real.1)
    )
} by {
    square_real_has_derivative_at(x0)
    cube_real_has_derivative_at(x0)
    derivative_pointwise_mul(square_real, cube_real, x0,
        x0 * Real.1 + x0 * Real.1,
        square_real(x0) * Real.1 + x0 * (x0 * Real.1 + x0 * Real.1))
    has_derivative_at(
        pointwise_mul(square_real, cube_real),
        x0,
        square_real(x0) * (square_real(x0) * Real.1 + x0 * (x0 * Real.1 + x0 * Real.1)) +
        cube_real(x0) * (x0 * Real.1 + x0 * Real.1)
    )
}

/// The product of the square and cube functions is differentiable at every point.
theorem differentiable_square_real_mul_cube_real(x0: Real) {
    differentiable_at(pointwise_mul(square_real, cube_real), x0)
} by {
    square_real_differentiable_at(x0)
    cube_real_differentiable_at(x0)
    differentiable_pointwise_mul(square_real, cube_real, x0)
    differentiable_at(pointwise_mul(square_real, cube_real), x0)
}

/// The product of the square function with itself has derivative from the product rule.
theorem derivative_square_real_mul_square_real(x0: Real) {
    has_derivative_at(
        pointwise_mul(square_real, square_real),
        x0,
        square_real(x0) * (x0 * Real.1 + x0 * Real.1) +
        square_real(x0) * (x0 * Real.1 + x0 * Real.1)
    )
} by {
    square_real_has_derivative_at(x0)
    derivative_pointwise_mul(square_real, square_real, x0,
        x0 * Real.1 + x0 * Real.1,
        x0 * Real.1 + x0 * Real.1)
    has_derivative_at(
        pointwise_mul(square_real, square_real),
        x0,
        square_real(x0) * (x0 * Real.1 + x0 * Real.1) +
        square_real(x0) * (x0 * Real.1 + x0 * Real.1)
    )
}

/// The product of the cube function with itself has derivative from the product rule.
theorem derivative_cube_real_mul_cube_real(x0: Real) {
    has_derivative_at(
        pointwise_mul(cube_real, cube_real),
        x0,
        cube_real(x0) * (square_real(x0) * Real.1 + x0 * (x0 * Real.1 + x0 * Real.1)) +
        cube_real(x0) * (square_real(x0) * Real.1 + x0 * (x0 * Real.1 + x0 * Real.1))
    )
} by {
    cube_real_has_derivative_at(x0)
    derivative_pointwise_mul(cube_real, cube_real, x0,
        square_real(x0) * Real.1 + x0 * (x0 * Real.1 + x0 * Real.1),
        square_real(x0) * Real.1 + x0 * (x0 * Real.1 + x0 * Real.1))
    has_derivative_at(
        pointwise_mul(cube_real, cube_real),
        x0,
        cube_real(x0) * (square_real(x0) * Real.1 + x0 * (x0 * Real.1 + x0 * Real.1)) +
        cube_real(x0) * (square_real(x0) * Real.1 + x0 * (x0 * Real.1 + x0 * Real.1))
    )
}

/// The product of the square function with itself is differentiable at every point.
theorem differentiable_square_real_mul_square_real(x0: Real) {
    differentiable_at(pointwise_mul(square_real, square_real), x0)
} by {
    square_real_differentiable_at(x0)
    differentiable_pointwise_mul(square_real, square_real, x0)
    differentiable_at(pointwise_mul(square_real, square_real), x0)
}

/// The product of the cube function with itself is differentiable at every point.
theorem differentiable_cube_real_mul_cube_real(x0: Real) {
    differentiable_at(pointwise_mul(cube_real, cube_real), x0)
} by {
    cube_real_differentiable_at(x0)
    differentiable_pointwise_mul(cube_real, cube_real, x0)
    differentiable_at(pointwise_mul(cube_real, cube_real), x0)
}
