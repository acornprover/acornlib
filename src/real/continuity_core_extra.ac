from data.basic.functions import compose, identity_fn
from data.basic.function_algebra import pointwise_add, pointwise_mul, pointwise_neg
from real.continuity_base import Real, continuous, continuous_at
from real.continuity_composition import continuous_imp_continuous_at,
    continuous_at_compose, continuous_compose, constant_function_is_continuous,
    constant_function_is_continuous_at
from real.continuity_sequences import identity_function_is_continuous
from real.continuity_pointwise import continuous_at_pointwise_neg,
    continuous_pointwise_neg, continuous_at_pointwise_add, continuous_pointwise_add,
    continuous_at_pointwise_sub, continuous_pointwise_sub
from real.continuity_pointwise_mul import continuous_at_pointwise_mul,
    continuous_pointwise_mul
from real.continuity_const_add import const_add_left, const_add_right,
    continuous_at_const_add_left, continuous_const_add_left,
    continuous_at_const_add_right, continuous_const_add_right
from real.continuity_const_mul import const_mul_left, const_mul_right,
    continuous_at_const_mul_left, continuous_const_mul_left,
    continuous_at_const_mul_right, continuous_const_mul_right

/// Alias for extracting pointwise continuity from a globally continuous function.
theorem continuous_at_of_continuous(f: Real -> Real, x: Real) {
    continuous(f) implies continuous_at(f, x)
} by {
    if continuous(f) {
        continuous_imp_continuous_at(f, x)
        continuous_at(f, x)
    }
}

/// The identity real function is continuous at every point.
theorem identity_function_is_continuous_at(x: Real) {
    continuous_at(identity_fn[Real], x)
} by {
    identity_function_is_continuous
    continuous_imp_continuous_at(identity_fn[Real], x)
    continuous_at(identity_fn[Real], x)
}

/// Constant real functions are continuous at every point, as a reusable endpoint.
theorem constant_function_continuous_at(c: Real, x: Real) {
    continuous_at(constant[Real, Real](c), x)
} by {
    constant_function_is_continuous_at(c, x)
}

/// Constant real functions are globally continuous, as a reusable endpoint.
theorem constant_function_continuous(c: Real) {
    continuous(constant[Real, Real](c))
} by {
    constant_function_is_continuous(c)
}

/// Globally continuous functions compose to a function continuous at each point.
theorem continuous_at_compose_of_continuous(f: Real -> Real, g: Real -> Real, x: Real) {
    continuous(f) and continuous(g) implies continuous_at(compose(f, g), x)
} by {
    if continuous(f) and continuous(g) {
        continuous_imp_continuous_at(g, x)
        continuous_at(g, x)
        continuous_imp_continuous_at(f, g(x))
        continuous_at(f, g(x))
        continuous_at_compose(f, g, x)
        continuous_at(compose(f, g), x)
    }
}

/// Alias for global continuity of a composition of continuous real functions.
theorem continuous_compose_of_continuous(f: Real -> Real, g: Real -> Real) {
    continuous(f) and continuous(g) implies continuous(compose(f, g))
} by {
    if continuous(f) and continuous(g) {
        continuous_compose(f, g)
        continuous(compose(f, g))
    }
}

/// A continuous real function has continuous pointwise negation at every point.
theorem continuous_at_pointwise_neg_of_continuous(f: Real -> Real, x: Real) {
    continuous(f) implies continuous_at(pointwise_neg(f), x)
} by {
    if continuous(f) {
        continuous_imp_continuous_at(f, x)
        continuous_at(f, x)
        continuous_at_pointwise_neg(f, x)
        continuous_at(pointwise_neg(f), x)
    }
}

/// Globally continuous functions have a pointwise sum continuous at every point.
theorem continuous_at_pointwise_add_of_continuous(f: Real -> Real, g: Real -> Real, x: Real) {
    continuous(f) and continuous(g) implies continuous_at(pointwise_add(f, g), x)
} by {
    if continuous(f) and continuous(g) {
        continuous_imp_continuous_at(f, x)
        continuous_at(f, x)
        continuous_imp_continuous_at(g, x)
        continuous_at(g, x)
        continuous_at_pointwise_add(f, g, x)
        continuous_at(pointwise_add(f, g), x)
    }
}

/// Globally continuous functions have a pointwise difference continuous at every point.
theorem continuous_at_pointwise_sub_of_continuous(f: Real -> Real, g: Real -> Real, x: Real) {
    continuous(f) and continuous(g) implies continuous_at(pointwise_add(f, pointwise_neg(g)), x)
} by {
    if continuous(f) and continuous(g) {
        continuous_imp_continuous_at(f, x)
        continuous_at(f, x)
        continuous_imp_continuous_at(g, x)
        continuous_at(g, x)
        continuous_at_pointwise_sub(f, g, x)
        continuous_at(pointwise_add(f, pointwise_neg(g)), x)
    }
}

/// Globally continuous functions have a pointwise product continuous at every point.
theorem continuous_at_pointwise_mul_of_continuous(f: Real -> Real, g: Real -> Real, x: Real) {
    continuous(f) and continuous(g) implies continuous_at(pointwise_mul[Real, Real](f, g), x)
} by {
    if continuous(f) and continuous(g) {
        continuous_imp_continuous_at(f, x)
        continuous_at(f, x)
        continuous_imp_continuous_at(g, x)
        continuous_at(g, x)
        continuous_at_pointwise_mul(f, g, x)
        continuous_at(pointwise_mul[Real, Real](f, g), x)
    }
}

/// A continuous real function has a continuous pointwise square at every point.
theorem continuous_at_pointwise_square(f: Real -> Real, x: Real) {
    continuous_at(f, x) implies continuous_at(pointwise_mul[Real, Real](f, f), x)
} by {
    if continuous_at(f, x) {
        continuous_at_pointwise_mul(f, f, x)
        continuous_at(pointwise_mul[Real, Real](f, f), x)
    }
}

/// A continuous real function has a globally continuous pointwise square.
theorem continuous_pointwise_square(f: Real -> Real) {
    continuous(f) implies continuous(pointwise_mul[Real, Real](f, f))
} by {
    if continuous(f) {
        continuous_pointwise_mul(f, f)
        continuous(pointwise_mul[Real, Real](f, f))
    }
}

/// Negation preserves global continuity, as a named closure endpoint.
theorem continuous_neg_of_continuous(f: Real -> Real) {
    continuous(f) implies continuous(pointwise_neg(f))
} by {
    if continuous(f) {
        continuous_pointwise_neg(f)
        continuous(pointwise_neg(f))
    }
}

/// Addition preserves global continuity, as a named closure endpoint.
theorem continuous_add_of_continuous(f: Real -> Real, g: Real -> Real) {
    continuous(f) and continuous(g) implies continuous(pointwise_add(f, g))
} by {
    if continuous(f) and continuous(g) {
        continuous_pointwise_add(f, g)
        continuous(pointwise_add(f, g))
    }
}

/// Subtraction preserves global continuity, as a named closure endpoint.
theorem continuous_sub_of_continuous(f: Real -> Real, g: Real -> Real) {
    continuous(f) and continuous(g) implies continuous(pointwise_add(f, pointwise_neg(g)))
} by {
    if continuous(f) and continuous(g) {
        continuous_pointwise_sub(f, g)
        continuous(pointwise_add(f, pointwise_neg(g)))
    }
}

/// Multiplication preserves global continuity, as a named closure endpoint.
theorem continuous_mul_of_continuous(f: Real -> Real, g: Real -> Real) {
    continuous(f) and continuous(g) implies continuous(pointwise_mul[Real, Real](f, g))
} by {
    if continuous(f) and continuous(g) {
        continuous_pointwise_mul(f, g)
        continuous(pointwise_mul[Real, Real](f, g))
    }
}

/// Adding a constant on the left to a continuous function is continuous at every point.
theorem continuous_at_const_add_left_of_continuous(c: Real, f: Real -> Real, x: Real) {
    continuous(f) implies continuous_at(const_add_left(c, f), x)
} by {
    if continuous(f) {
        continuous_imp_continuous_at(f, x)
        continuous_at(f, x)
        continuous_at_const_add_left(c, f, x)
        continuous_at(const_add_left(c, f), x)
    }
}

/// Adding a constant on the right to a continuous function is continuous at every point.
theorem continuous_at_const_add_right_of_continuous(f: Real -> Real, c: Real, x: Real) {
    continuous(f) implies continuous_at(const_add_right(f, c), x)
} by {
    if continuous(f) {
        continuous_imp_continuous_at(f, x)
        continuous_at(f, x)
        continuous_at_const_add_right(f, c, x)
        continuous_at(const_add_right(f, c), x)
    }
}

/// Multiplying a continuous function by a constant on the left is continuous at every point.
theorem continuous_at_const_mul_left_of_continuous(c: Real, f: Real -> Real, x: Real) {
    continuous(f) implies continuous_at(const_mul_left(c, f), x)
} by {
    if continuous(f) {
        continuous_imp_continuous_at(f, x)
        continuous_at(f, x)
        continuous_at_const_mul_left(c, f, x)
        continuous_at(const_mul_left(c, f), x)
    }
}

/// Multiplying a continuous function by a constant on the right is continuous at every point.
theorem continuous_at_const_mul_right_of_continuous(f: Real -> Real, c: Real, x: Real) {
    continuous(f) implies continuous_at(const_mul_right(f, c), x)
} by {
    if continuous(f) {
        continuous_imp_continuous_at(f, x)
        continuous_at(f, x)
        continuous_at_const_mul_right(f, c, x)
        continuous_at(const_mul_right(f, c), x)
    }
}

/// Adding a constant on the left preserves global continuity, as a named closure endpoint.
theorem continuous_const_add_left_of_continuous(c: Real, f: Real -> Real) {
    continuous(f) implies continuous(const_add_left(c, f))
} by {
    if continuous(f) {
        continuous_const_add_left(c, f)
        continuous(const_add_left(c, f))
    }
}

/// Adding a constant on the right preserves global continuity, as a named closure endpoint.
theorem continuous_const_add_right_of_continuous(f: Real -> Real, c: Real) {
    continuous(f) implies continuous(const_add_right(f, c))
} by {
    if continuous(f) {
        continuous_const_add_right(f, c)
        continuous(const_add_right(f, c))
    }
}

/// Multiplication by a constant on the left preserves global continuity, as a named closure endpoint.
theorem continuous_const_mul_left_of_continuous(c: Real, f: Real -> Real) {
    continuous(f) implies continuous(const_mul_left(c, f))
} by {
    if continuous(f) {
        continuous_const_mul_left(c, f)
        continuous(const_mul_left(c, f))
    }
}

/// Multiplication by a constant on the right preserves global continuity, as a named closure endpoint.
theorem continuous_const_mul_right_of_continuous(f: Real -> Real, c: Real) {
    continuous(f) implies continuous(const_mul_right(f, c))
} by {
    if continuous(f) {
        continuous_const_mul_right(f, c)
        continuous(const_mul_right(f, c))
    }
}
