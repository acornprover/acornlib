from order import lt_imp_lte, lte_trans
from order_set import closed_interval_set, closed_interval_set_lower_le,
    closed_interval_set_le_upper
from order import closed_interval
from real.continuity_base import Real, continuous
from real.intermediate_value import intermediate_value_closed_interval

numerals Real

/// A point of an ordered subinterval belongs to the ambient interval.
theorem closed_interval_contains_ordered_subinterval_point(
    lower: Real, upper: Real, left: Real, right: Real, point: Real
) {
    lower <= left and right <= upper and closed_interval_set(left, right).contains(point)
    implies closed_interval_set(lower, upper).contains(point)
} by {
    if lower <= left and right <= upper and closed_interval_set(left, right).contains(point) {
        closed_interval_set_lower_le(left, right, point)
        left <= point
        lte_trans[Real](lower, left, point)
        lower <= point
        closed_interval_set_le_upper(left, right, point)
        point <= right
        lte_trans[Real](point, right, upper)
        point <= upper
        closed_interval(lower, upper, point)
        closed_interval_set(lower, upper).contains(point)
    }
}

/// Transport a subinterval value witness to the ambient interval.
theorem ordered_subinterval_exists_value_transport(
    f: Real -> Real, lower: Real, upper: Real, left: Real, right: Real, target: Real
) {
    lower <= left and right <= upper and exists(point: Real) {
        closed_interval_set(left, right).contains(point) and f(point) = target
    }
    implies exists(point: Real) {
        closed_interval_set(lower, upper).contains(point) and f(point) = target
    }
} by {
    if lower <= left and right <= upper and exists(point: Real) {
        closed_interval_set(left, right).contains(point) and f(point) = target
    } {
        let point: Real satisfy {
            closed_interval_set(left, right).contains(point) and f(point) = target
        }
        closed_interval_set(left, right).contains(point)
        closed_interval_contains_ordered_subinterval_point(lower, upper, left, right, point)
        closed_interval_set(lower, upper).contains(point)
        f(point) = target
        exists(root: Real) {
            closed_interval_set(lower, upper).contains(root) and f(root) = target
        }
    }
}

/// A continuous real function with a sign change from nonpositive to nonnegative has a zero.
theorem bolzano_closed_interval_nonpositive_to_nonnegative(
    f: Real -> Real, lower: Real, upper: Real
) {
    continuous(f) and lower <= upper and f(lower) <= Real.0 and Real.0 <= f(upper)
    implies exists(root: Real) {
        closed_interval_set(lower, upper).contains(root) and f(root) = Real.0
    }
} by {
    if continuous(f) and lower <= upper and f(lower) <= Real.0 and Real.0 <= f(upper) {
        intermediate_value_closed_interval(f, lower, upper, Real.0)
        let root: Real satisfy {
            closed_interval_set(lower, upper).contains(root) and f(root) = Real.0
        }
        exists(point: Real) {
            closed_interval_set(lower, upper).contains(point) and f(point) = Real.0
        }
    }
}

/// A continuous real function changing from negative to positive has a zero.
theorem bolzano_closed_interval_negative_to_positive(
    f: Real -> Real, lower: Real, upper: Real
) {
    continuous(f) and lower <= upper and f(lower) < Real.0 and Real.0 < f(upper)
    implies exists(root: Real) {
        closed_interval_set(lower, upper).contains(root) and f(root) = Real.0
    }
} by {
    if continuous(f) and lower <= upper and f(lower) < Real.0 and Real.0 < f(upper) {
        lt_imp_lte(f(lower), Real.0)
        f(lower) <= Real.0
        lt_imp_lte(Real.0, f(upper))
        Real.0 <= f(upper)
        bolzano_closed_interval_nonpositive_to_nonnegative(f, lower, upper)
        let root: Real satisfy {
            closed_interval_set(lower, upper).contains(root) and f(root) = Real.0
        }
        exists(point: Real) {
            closed_interval_set(lower, upper).contains(point) and f(point) = Real.0
        }
    }
}

/// Strict endpoint inequalities give an IVT witness for the target value.
theorem intermediate_value_closed_interval_strict_between(
    f: Real -> Real, lower: Real, upper: Real, target: Real
) {
    continuous(f) and f(lower) < target and target < f(upper) and lower <= upper
    implies exists(point: Real) {
        closed_interval_set(lower, upper).contains(point) and f(point) = target
    }
} by {
    if continuous(f) and f(lower) < target and target < f(upper) and lower <= upper {
        lt_imp_lte(f(lower), target)
        f(lower) <= target
        lt_imp_lte(target, f(upper))
        target <= f(upper)
        intermediate_value_closed_interval(f, lower, upper, target)
        let point: Real satisfy {
            closed_interval_set(lower, upper).contains(point) and f(point) = target
        }
        exists(root: Real) {
            closed_interval_set(lower, upper).contains(root) and f(root) = target
        }
    }
}
