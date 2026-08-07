from data.basic.set import Set, compl_contains_eq, difference_contains_eq, difference_contains_intro,
    double_inclusion, singleton_contains_eq, subset_contains, subset_contains_eq,
    sets_subset_contain_union, sets_subset_union, union_contains_eq,
    union_contains_left, union_contains_right
from real.real_field import Real
from real.real_base import lt_lte_trans, min_lte_left, min_lte_right, min_pos_pos
from real.topology import adherent_point_eps, adherent_point_intro, closure, closure_contains_eq,
    eps_adherent_of_contains_close, eps_adherent_of_subset, is_adherent_point_of_set,
    is_eps_adherent_to_set, is_closed_set, is_isolated_point_of_set,
    is_limit_point_of_set
from real.topology_complements import not_adherent_has_missing_eps
from real.topology_difference import adherent_difference_left

/// True if a real number is a limit point of a set.
define derived_set_contains(s: Set[Real], x: Real) -> Bool {
    is_limit_point_of_set(s, x)
}

/// The set of limit points of a real set.
define derived_set(s: Set[Real]) -> Set[Real] {
    Set[Real].new(derived_set_contains(s))
}

/// Membership in the derived set is limit-point membership.
theorem derived_set_contains_eq(s: Set[Real], x: Real) {
    derived_set(s).contains(x) = is_limit_point_of_set(s, x)
}

/// A limit point of a real set is adherent to that set.
theorem limit_point_is_adherent(s: Set[Real], x: Real) {
    is_limit_point_of_set(s, x) implies is_adherent_point_of_set(s, x)
} by {
    if is_limit_point_of_set(s, x) {
        is_limit_point_of_set(s, x) = is_adherent_point_of_set(s.difference(Set[Real].singleton(x)), x)
        is_adherent_point_of_set(s.difference(Set[Real].singleton(x)), x)
        adherent_difference_left(s, Set[Real].singleton(x), x)
        is_adherent_point_of_set(s, x)
    }
}

/// A limit point of a real set lies in its closure.
theorem limit_point_in_closure(s: Set[Real], x: Real) {
    is_limit_point_of_set(s, x) implies closure(s).contains(x)
} by {
    if is_limit_point_of_set(s, x) {
        limit_point_is_adherent(s, x)
        is_adherent_point_of_set(s, x)
        closure_contains_eq(s, x)
        closure(s).contains(x)
    }
}

/// A derived-set member lies in the closure.
theorem derived_set_member_in_closure(s: Set[Real], x: Real) {
    derived_set(s).contains(x) implies closure(s).contains(x)
} by {
    if derived_set(s).contains(x) {
        derived_set_contains_eq(s, x)
        is_limit_point_of_set(s, x)
        limit_point_in_closure(s, x)
        closure(s).contains(x)
    }
}

/// The derived set is contained in the closure.
theorem derived_set_subset_closure(s: Set[Real]) {
    derived_set(s).subset(closure(s))
} by {
    forall(x: Real) {
        if derived_set(s).contains(x) {
            derived_set_member_in_closure(s, x)
        }
    }
}

/// A closed real set contains its limit points.
theorem closed_set_contains_limit_point(s: Set[Real], x: Real) {
    is_closed_set(s) and is_limit_point_of_set(s, x) implies s.contains(x)
} by {
    if is_closed_set(s) and is_limit_point_of_set(s, x) {
        limit_point_is_adherent(s, x)
        is_adherent_point_of_set(s, x)
        is_closed_set(s) = forall(y: Real) {
            is_adherent_point_of_set(s, y) implies s.contains(y)
        }
        s.contains(x)
    }
}

/// A closed real set contains its derived set.
theorem derived_set_subset_of_closed_set(s: Set[Real]) {
    is_closed_set(s) implies derived_set(s).subset(s)
} by {
    if is_closed_set(s) {
        forall(x: Real) {
            if derived_set(s).contains(x) {
                derived_set_contains_eq(s, x)
                is_limit_point_of_set(s, x)
                closed_set_contains_limit_point(s, x)
                s.contains(x)
            }
        }
    }
}

/// Difference by a singleton is monotone in the left argument.
theorem singleton_difference_subset_of_subset(s: Set[Real], t: Set[Real], x: Real) {
    s.subset(t) implies s.difference(Set[Real].singleton(x)).subset(t.difference(Set[Real].singleton(x)))
} by {
    if s.subset(t) {
        forall(y: Real) {
            if s.difference(Set[Real].singleton(x)).contains(y) {
                difference_contains_eq(s, Set[Real].singleton(x), y)
                s.contains(y)
                not Set[Real].singleton(x).contains(y)
                subset_contains(s, t, y)
                t.contains(y)
                difference_contains_intro[Real](t, Set[Real].singleton(x), y)
                t.difference(Set[Real].singleton(x)).contains(y)
            }
        }
    }
}

/// Limit-point membership is monotone under set inclusion.
theorem limit_point_of_subset(s: Set[Real], t: Set[Real], x: Real) {
    s.subset(t) and is_limit_point_of_set(s, x) implies is_limit_point_of_set(t, x)
} by {
    if s.subset(t) and is_limit_point_of_set(s, x) {
        is_limit_point_of_set(s, x) = is_adherent_point_of_set(s.difference(Set[Real].singleton(x)), x)
        is_adherent_point_of_set(s.difference(Set[Real].singleton(x)), x)
        singleton_difference_subset_of_subset(s, t, x)
        s.difference(Set[Real].singleton(x)).subset(t.difference(Set[Real].singleton(x)))
        forall(eps: Real) {
            if eps.is_positive {
                adherent_point_eps(s.difference(Set[Real].singleton(x)), x, eps)
                is_eps_adherent_to_set(s.difference(Set[Real].singleton(x)), x, eps)
                eps_adherent_of_subset(s.difference(Set[Real].singleton(x)),
                    t.difference(Set[Real].singleton(x)), x, eps)
                is_eps_adherent_to_set(t.difference(Set[Real].singleton(x)), x, eps)
            }
        }
        adherent_point_intro(t.difference(Set[Real].singleton(x)), x)
        is_adherent_point_of_set(t.difference(Set[Real].singleton(x)), x)
        is_limit_point_of_set(t, x)
    }
}

/// The derived-set operation is monotone under set inclusion.
theorem derived_set_mono(s: Set[Real], t: Set[Real]) {
    s.subset(t) implies derived_set(s).subset(derived_set(t))
} by {
    if s.subset(t) {
        forall(x: Real) {
            if derived_set(s).contains(x) {
                derived_set_contains_eq(s, x)
                is_limit_point_of_set(s, x)
                limit_point_of_subset(s, t, x)
                is_limit_point_of_set(t, x)
                derived_set_contains_eq(t, x)
                derived_set(t).contains(x)
            }
        }
    }
}

/// An isolated point of a real set is not a limit point of that set.
theorem isolated_point_not_limit_point(s: Set[Real], x: Real) {
    is_isolated_point_of_set(s, x) implies not is_limit_point_of_set(s, x)
} by {
    if is_isolated_point_of_set(s, x) {
        is_isolated_point_of_set(s, x) = s.contains(x) and exists(eps: Real) {
            eps.is_positive and forall(y: Real) {
                s.contains(y) and y != x implies not y.is_close(x, eps)
            }
        }
        let eps: Real satisfy {
            eps.is_positive and forall(y: Real) {
                s.contains(y) and y != x implies not y.is_close(x, eps)
            }
        }
        if is_limit_point_of_set(s, x) {
            is_limit_point_of_set(s, x) = is_adherent_point_of_set(s.difference(Set[Real].singleton(x)), x)
            is_adherent_point_of_set(s.difference(Set[Real].singleton(x)), x)
            adherent_point_eps(s.difference(Set[Real].singleton(x)), x, eps)
            is_eps_adherent_to_set(s.difference(Set[Real].singleton(x)), x, eps)
            let y: Real satisfy {
                s.difference(Set[Real].singleton(x)).contains(y) and y.is_close(x, eps)
            }
            difference_contains_eq(s, Set[Real].singleton(x), y)
            s.contains(y)
            not Set[Real].singleton(x).contains(y)
            singleton_contains_eq[Real](x, y)
            y != x
            not y.is_close(x, eps)
            y.is_close(x, eps)
            false
        }
    }
}

/// An isolated point of a real set is not in its derived set.
theorem isolated_point_not_in_derived_set(s: Set[Real], x: Real) {
    is_isolated_point_of_set(s, x) implies not derived_set(s).contains(x)
} by {
    if is_isolated_point_of_set(s, x) {
        isolated_point_not_limit_point(s, x)
        if derived_set(s).contains(x) {
            derived_set_contains_eq(s, x)
            is_limit_point_of_set(s, x)
            false
        }
    }
}

/// A point of the derived set is not isolated.
theorem derived_set_member_not_isolated(s: Set[Real], x: Real) {
    derived_set(s).contains(x) implies not is_isolated_point_of_set(s, x)
} by {
    if derived_set(s).contains(x) {
        if is_isolated_point_of_set(s, x) {
            isolated_point_not_in_derived_set(s, x)
            false
        }
    }
}

/// A limit point of a real set is not isolated in that set.
theorem limit_point_not_isolated(s: Set[Real], x: Real) {
    is_limit_point_of_set(s, x) implies not is_isolated_point_of_set(s, x)
} by {
    if is_limit_point_of_set(s, x) {
        derived_set_contains_eq(s, x)
        derived_set(s).contains(x)
        derived_set_member_not_isolated(s, x)
        not is_isolated_point_of_set(s, x)
    }
}

/// The complement of the derived set contains every isolated point.
theorem isolated_point_in_derived_set_complement(s: Set[Real], x: Real) {
    is_isolated_point_of_set(s, x) implies derived_set(s).c.contains(x)
} by {
    if is_isolated_point_of_set(s, x) {
        isolated_point_not_in_derived_set(s, x)
        not derived_set(s).contains(x)
        compl_contains_eq(derived_set(s), x)
        derived_set(s).c.contains(x)
    }
}

/// A derived-set member of a subset is adherent to any superset.
theorem derived_set_member_adherent_to_superset(s: Set[Real], t: Set[Real], x: Real) {
    s.subset(t) and derived_set(s).contains(x) implies is_adherent_point_of_set(t, x)
} by {
    if s.subset(t) and derived_set(s).contains(x) {
        derived_set_contains_eq(s, x)
        is_limit_point_of_set(s, x)
        limit_point_of_subset(s, t, x)
        is_limit_point_of_set(t, x)
        limit_point_is_adherent(t, x)
        is_adherent_point_of_set(t, x)
    }
}

/// Closeness at a smaller radius implies closeness at any larger radius.
theorem close_of_close_le(a: Real, b: Real, eps: Real, delta: Real) {
    a.is_close(b, eps) and eps <= delta implies a.is_close(b, delta)
} by {
    if a.is_close(b, eps) and eps <= delta {
        a.is_close(b, eps) = (a - b).abs < eps
        (a - b).abs < eps
        lt_lte_trans((a - b).abs, eps, delta)
        (a - b).abs < delta
        a.is_close(b, delta) = (a - b).abs < delta
        a.is_close(b, delta)
    }
}

/// A point of the punctured union lies in one of the two punctured summands.
theorem union_singleton_difference_member_cases(s: Set[Real], t: Set[Real], x: Real, y: Real) {
    s.union(t).difference(Set[Real].singleton(x)).contains(y) implies
        s.difference(Set[Real].singleton(x)).contains(y) or
        t.difference(Set[Real].singleton(x)).contains(y)
} by {
    if s.union(t).difference(Set[Real].singleton(x)).contains(y) {
        difference_contains_eq(s.union(t), Set[Real].singleton(x), y)
        s.union(t).contains(y)
        not Set[Real].singleton(x).contains(y)
        union_contains_eq(s, t, y)
        if s.contains(y) {
            difference_contains_intro[Real](s, Set[Real].singleton(x), y)
            s.difference(Set[Real].singleton(x)).contains(y)
            s.difference(Set[Real].singleton(x)).contains(y) or
                t.difference(Set[Real].singleton(x)).contains(y)
        } else {
            t.contains(y)
            difference_contains_intro[Real](t, Set[Real].singleton(x), y)
            t.difference(Set[Real].singleton(x)).contains(y)
            s.difference(Set[Real].singleton(x)).contains(y) or
                t.difference(Set[Real].singleton(x)).contains(y)
        }
    }
}

/// If both punctured summands miss positive radii, the punctured union misses
/// the minimum radius. This is the radius step for the hard derived-union inclusion.
theorem missing_eps_union_diff_of_missing_parts(
    s: Set[Real], t: Set[Real], x: Real, eps_s: Real, eps_t: Real
) {
    eps_s.is_positive and eps_t.is_positive and
    not is_eps_adherent_to_set(s.difference(Set[Real].singleton(x)), x, eps_s) and
    not is_eps_adherent_to_set(t.difference(Set[Real].singleton(x)), x, eps_t)
    implies
    not is_eps_adherent_to_set(s.union(t).difference(Set[Real].singleton(x)), x, eps_s.min(eps_t))
} by {
    if eps_s.is_positive and eps_t.is_positive and
        not is_eps_adherent_to_set(s.difference(Set[Real].singleton(x)), x, eps_s) and
        not is_eps_adherent_to_set(t.difference(Set[Real].singleton(x)), x, eps_t) {
        let eps = eps_s.min(eps_t)
        min_pos_pos(eps_s, eps_t)
        eps.is_positive
        min_lte_left(eps_s, eps_t)
        eps <= eps_s
        min_lte_right(eps_s, eps_t)
        eps <= eps_t
        if is_eps_adherent_to_set(s.union(t).difference(Set[Real].singleton(x)), x, eps) {
            let y: Real satisfy {
                s.union(t).difference(Set[Real].singleton(x)).contains(y) and y.is_close(x, eps)
            }
            union_singleton_difference_member_cases(s, t, x, y)
            s.difference(Set[Real].singleton(x)).contains(y) or
                t.difference(Set[Real].singleton(x)).contains(y)
            if s.difference(Set[Real].singleton(x)).contains(y) {
                close_of_close_le(y, x, eps, eps_s)
                y.is_close(x, eps_s)
                eps_adherent_of_contains_close(s.difference(Set[Real].singleton(x)), x, y, eps_s)
                is_eps_adherent_to_set(s.difference(Set[Real].singleton(x)), x, eps_s)
                false
            } else {
                t.difference(Set[Real].singleton(x)).contains(y)
                close_of_close_le(y, x, eps, eps_t)
                y.is_close(x, eps_t)
                eps_adherent_of_contains_close(t.difference(Set[Real].singleton(x)), x, y, eps_t)
                is_eps_adherent_to_set(t.difference(Set[Real].singleton(x)), x, eps_t)
                false
            }
        }
        not is_eps_adherent_to_set(s.union(t).difference(Set[Real].singleton(x)), x, eps)
        eps = eps_s.min(eps_t)
        not is_eps_adherent_to_set(s.union(t).difference(Set[Real].singleton(x)), x, eps_s.min(eps_t))
    }
}

/// A limit point of the left summand is a limit point of the union.
theorem limit_point_union_left(s: Set[Real], t: Set[Real], x: Real) {
    is_limit_point_of_set(s, x) implies is_limit_point_of_set(s.union(t), x)
} by {
    if is_limit_point_of_set(s, x) {
        sets_subset_union[Real](s, t)
        s.subset(s.union(t))
        limit_point_of_subset(s, s.union(t), x)
        is_limit_point_of_set(s.union(t), x)
    }
}

/// A limit point of the right summand is a limit point of the union.
theorem limit_point_union_right(s: Set[Real], t: Set[Real], x: Real) {
    is_limit_point_of_set(t, x) implies is_limit_point_of_set(s.union(t), x)
} by {
    if is_limit_point_of_set(t, x) {
        sets_subset_union[Real](s, t)
        t.subset(s.union(t))
        limit_point_of_subset(t, s.union(t), x)
        is_limit_point_of_set(s.union(t), x)
    }
}

/// A limit point of a binary union is a limit point of at least one summand.
theorem limit_point_union_cases(s: Set[Real], t: Set[Real], x: Real) {
    is_limit_point_of_set(s.union(t), x) implies
        is_limit_point_of_set(s, x) or is_limit_point_of_set(t, x)
} by {
    if is_limit_point_of_set(s.union(t), x) {
        if is_limit_point_of_set(s, x) {
            is_limit_point_of_set(s, x) or is_limit_point_of_set(t, x)
        } else {
            if is_limit_point_of_set(t, x) {
                is_limit_point_of_set(s, x) or is_limit_point_of_set(t, x)
            } else {
                is_limit_point_of_set(s, x) =
                    is_adherent_point_of_set(s.difference(Set[Real].singleton(x)), x)
                not is_adherent_point_of_set(s.difference(Set[Real].singleton(x)), x)
                not_adherent_has_missing_eps(s.difference(Set[Real].singleton(x)), x)
                let eps_s: Real satisfy {
                    eps_s.is_positive and
                    not is_eps_adherent_to_set(s.difference(Set[Real].singleton(x)), x, eps_s)
                }
                is_limit_point_of_set(t, x) =
                    is_adherent_point_of_set(t.difference(Set[Real].singleton(x)), x)
                not is_adherent_point_of_set(t.difference(Set[Real].singleton(x)), x)
                not_adherent_has_missing_eps(t.difference(Set[Real].singleton(x)), x)
                let eps_t: Real satisfy {
                    eps_t.is_positive and
                    not is_eps_adherent_to_set(t.difference(Set[Real].singleton(x)), x, eps_t)
                }
                missing_eps_union_diff_of_missing_parts(s, t, x, eps_s, eps_t)
                let eps = eps_s.min(eps_t)
                min_pos_pos(eps_s, eps_t)
                eps.is_positive
                not is_eps_adherent_to_set(s.union(t).difference(Set[Real].singleton(x)), x, eps)
                is_limit_point_of_set(s.union(t), x) =
                    is_adherent_point_of_set(s.union(t).difference(Set[Real].singleton(x)), x)
                is_adherent_point_of_set(s.union(t).difference(Set[Real].singleton(x)), x)
                adherent_point_eps(s.union(t).difference(Set[Real].singleton(x)), x, eps)
                is_eps_adherent_to_set(s.union(t).difference(Set[Real].singleton(x)), x, eps)
                false
            }
        }
    }
}

/// The union of the two derived sets is contained in the derived set of the union.
theorem derived_set_union_superset(s: Set[Real], t: Set[Real]) {
    derived_set(s).union(derived_set(t)).subset(derived_set(s.union(t)))
} by {
    sets_subset_union[Real](s, t)
    s.subset(s.union(t))
    t.subset(s.union(t))
    derived_set_mono(s, s.union(t))
    derived_set(s).subset(derived_set(s.union(t)))
    derived_set_mono(t, s.union(t))
    derived_set(t).subset(derived_set(s.union(t)))
    sets_subset_contain_union[Real](derived_set(s), derived_set(t), derived_set(s.union(t)))
}

/// A member of the derived set of a union lies in one of the two derived sets.
theorem derived_set_union_member_subset(s: Set[Real], t: Set[Real], x: Real) {
    derived_set(s.union(t)).contains(x) implies derived_set(s).union(derived_set(t)).contains(x)
} by {
    if derived_set(s.union(t)).contains(x) {
        derived_set_contains_eq(s.union(t), x)
        is_limit_point_of_set(s.union(t), x)
        limit_point_union_cases(s, t, x)
        is_limit_point_of_set(s, x) or is_limit_point_of_set(t, x)
        if is_limit_point_of_set(s, x) {
            derived_set_contains_eq(s, x)
            derived_set(s).contains(x)
            union_contains_left(derived_set(s), derived_set(t), x)
            derived_set(s).union(derived_set(t)).contains(x)
        } else {
            is_limit_point_of_set(t, x)
            derived_set_contains_eq(t, x)
            derived_set(t).contains(x)
            union_contains_right(derived_set(s), derived_set(t), x)
            derived_set(s).union(derived_set(t)).contains(x)
        }
    }
}

/// The derived set of the union is contained in the union of the derived sets.
theorem derived_set_union_subset(s: Set[Real], t: Set[Real]) {
    derived_set(s.union(t)).subset(derived_set(s).union(derived_set(t)))
} by {
    let lhs = derived_set(s.union(t))
    let rhs = derived_set(s).union(derived_set(t))
    if not lhs.subset(rhs) {
        subset_contains_eq[Real](lhs, rhs)
        let x: Real satisfy {
            lhs.contains(x) and not rhs.contains(x)
        }
        derived_set_union_member_subset(s, t, x)
        rhs.contains(x)
        false
    }
}

/// Derived sets distribute over binary union on the real line.
theorem derived_set_union_eq(s: Set[Real], t: Set[Real]) {
    derived_set(s.union(t)) = derived_set(s).union(derived_set(t))
} by {
    derived_set_union_subset(s, t)
    derived_set_union_superset(s, t)
    derived_set(s.union(t)).subset(derived_set(s).union(derived_set(t)))
    derived_set(s).union(derived_set(t)).subset(derived_set(s.union(t)))
    double_inclusion(derived_set(s.union(t)), derived_set(s).union(derived_set(t)))
}
