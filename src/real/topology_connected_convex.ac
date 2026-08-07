from order import lte_antisymm, lte_trans, not_lt_imp_gte
from order_set import closed_interval_set, closed_interval_set_contains_eq,
    left_open_interval_set, left_open_interval_set_contains_eq,
    open_interval_set, open_interval_set_contains_eq,
    right_open_interval_set, right_open_interval_set_contains_eq
from order import closed_interval, left_open_interval, open_interval, right_open_interval
from real.real_field import Real
from data.basic.set import Set
from real.topology import disconnecting_triple, is_connected_real_set
from real.topology_connected_algebra import connected_real_set_contains_between

/// True if a real set contains every point between any two of its points.
define is_order_convex_real_set(s: Set[Real]) -> Bool {
    forall(x: Real, y: Real, z: Real) {
        s.contains(x) and s.contains(y) and x <= z and z <= y implies s.contains(z)
    }
}

/// An order-convex real set contains points between two of its points.
theorem order_convex_real_set_contains_between(s: Set[Real], x: Real, y: Real, z: Real) {
    is_order_convex_real_set(s) and s.contains(x) and s.contains(y) and x <= z and z <= y
    implies s.contains(z)
} by {
    if is_order_convex_real_set(s) and s.contains(x) and s.contains(y) and x <= z and z <= y {
        is_order_convex_real_set(s) = forall(a: Real, b: Real, c: Real) {
            s.contains(a) and s.contains(b) and a <= c and c <= b implies s.contains(c)
        }
        s.contains(x) and s.contains(y) and x <= z and z <= y implies s.contains(z)
        s.contains(z)
    }
}

/// A connected real set contains points weakly between two of its points.
theorem connected_real_set_contains_between_le(s: Set[Real], x: Real, y: Real, z: Real) {
    is_connected_real_set(s) and s.contains(x) and s.contains(y) and x <= z and z <= y
    implies s.contains(z)
} by {
    if is_connected_real_set(s) and s.contains(x) and s.contains(y) and x <= z and z <= y {
        if x < y {
            connected_real_set_contains_between(s, x, y, z)
            s.contains(z)
        } else {
            lte_trans[Real](x, z, y)
            x <= y
            not_lt_imp_gte[Real](x, y)
            x >= y
            y <= x
            lte_antisymm(x, y)
            x = y
            z <= x
            lte_antisymm(x, z)
            x = z
            s.contains(z)
        }
    }
}

/// Every order-convex real set is connected.
theorem order_convex_real_set_is_connected(s: Set[Real]) {
    is_order_convex_real_set(s) implies is_connected_real_set(s)
} by {
    if is_order_convex_real_set(s) {
        forall(x: Real, y: Real, z: Real) {
            if disconnecting_triple(s, x, y, z) {
                disconnecting_triple(s, x, y, z) =
                    (s.contains(x) and s.contains(y) and x < y and x <= z and z <= y and not s.contains(z))
                s.contains(x)
                s.contains(y)
                x <= z
                z <= y
                order_convex_real_set_contains_between(s, x, y, z)
                s.contains(z)
                not s.contains(z)
                false
            }
        }
        is_connected_real_set(s)
    }
}

/// A connected real set contains the closed interval between any two of its points.
theorem connected_real_set_contains_closed_interval_between(s: Set[Real], x: Real, y: Real) {
    is_connected_real_set(s) and s.contains(x) and s.contains(y) and x <= y
    implies closed_interval_set(x, y).subset(s)
} by {
    if is_connected_real_set(s) and s.contains(x) and s.contains(y) and x <= y {
        forall(z: Real) {
            if closed_interval_set(x, y).contains(z) {
                closed_interval_set_contains_eq(x, y, z)
                closed_interval(x, y, z)
                x <= z
                z <= y
                connected_real_set_contains_between_le(s, x, y, z)
                s.contains(z)
            }
        }
    }
}

/// A connected real set contains the open interval between any two of its points.
theorem connected_real_set_contains_open_interval_between(s: Set[Real], x: Real, y: Real) {
    is_connected_real_set(s) and s.contains(x) and s.contains(y) and x <= y
    implies open_interval_set(x, y).subset(s)
} by {
    if is_connected_real_set(s) and s.contains(x) and s.contains(y) and x <= y {
        forall(z: Real) {
            if open_interval_set(x, y).contains(z) {
                open_interval_set_contains_eq(x, y, z)
                open_interval(x, y, z)
                x < z
                z < y
                x <= z
                z <= y
                connected_real_set_contains_between_le(s, x, y, z)
                s.contains(z)
            }
        }
    }
}

/// A connected real set contains the left-open interval between any two of its points.
theorem connected_real_set_contains_left_open_interval_between(s: Set[Real], x: Real, y: Real) {
    is_connected_real_set(s) and s.contains(x) and s.contains(y) and x <= y
    implies left_open_interval_set(x, y).subset(s)
} by {
    if is_connected_real_set(s) and s.contains(x) and s.contains(y) and x <= y {
        forall(z: Real) {
            if left_open_interval_set(x, y).contains(z) {
                left_open_interval_set_contains_eq(x, y, z)
                left_open_interval(x, y, z)
                x < z
                z <= y
                x <= z
                connected_real_set_contains_between_le(s, x, y, z)
                s.contains(z)
            }
        }
    }
}

/// A connected real set contains the right-open interval between any two of its points.
theorem connected_real_set_contains_right_open_interval_between(s: Set[Real], x: Real, y: Real) {
    is_connected_real_set(s) and s.contains(x) and s.contains(y) and x <= y
    implies right_open_interval_set(x, y).subset(s)
} by {
    if is_connected_real_set(s) and s.contains(x) and s.contains(y) and x <= y {
        forall(z: Real) {
            if right_open_interval_set(x, y).contains(z) {
                right_open_interval_set_contains_eq(x, y, z)
                right_open_interval(x, y, z)
                x <= z
                z < y
                z <= y
                connected_real_set_contains_between_le(s, x, y, z)
                s.contains(z)
            }
        }
    }
}
