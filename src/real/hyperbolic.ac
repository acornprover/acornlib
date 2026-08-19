/// Hyperbolic functions: sine, cosine, tangent, and the inverse hyperbolic
/// sine.
///
/// The hyperbolic sine and cosine are defined from the real exponential by the
/// classical formulas
///     sinh(x) = (e^x - e^(-x)) / 2,   cosh(x) = (e^x + e^(-x)) / 2,
/// and the hyperbolic tangent is their quotient.  Since Real.exp is differentiable
/// with derivative itself and (-x).exp has derivative -(-x).exp (chain rule with
/// the affine negation), the derivative rules of derivative_rules.ac and
/// derivative_quotient.ac give sinh' = cosh and cosh' = sinh exactly as
/// tan' = sec^2 was obtained in inverse_trig.ac.
///
/// The main results are:
/// - the identity cosh^2 - sinh^2 = 1 (`cosh_sq_sub_sinh_sq`),
/// - the derivatives (`sinh_has_derivative_at`, `cosh_has_derivative_at`,
///   `sinh_is_derivative_fn`, `cosh_is_derivative_fn`),
/// - the addition formulas (`sinh_add`, `cosh_add`),
/// - the parity laws (`sinh_neg`, `cosh_neg`),
/// - strict monotonicity and injectivity of sinh (`sinh_strictly_increasing`,
///   `sinh_injective`), and surjectivity (`exists_sinh_of`) via the
///   intermediate value theorem from continuity, unboundedness and monotonicity,
/// - the inverse hyperbolic sine (`is_arsinh`, `arsinh`) with the left inverse
///   law `sinh_arsinh`, and the closed form arsinh(x) = ln(x + (x^2 + 1).sqrt)
///   (`arsinh_log_form`).

from nat import Nat
from order import lte_antisymm, lte_trans, lt_trans, lt_imp_lte, not_lt_imp_gte, lt_of_lt_of_lte, lt_imp_ne, lt_imp_ne_symm, not_lt_self, not_lte_imp_gt, lt_of_lte_of_lt, lte_self, lt_of_lte_of_ne
from real.real_field import Real, mul_div, mul_inverse, inverse_div, div_mul_cancel_left, mul_div_cancel, prod_eq_to_div_eq, zero_is_different_than_one, div_cancel_common
from real.real_base import add_comm, add_assoc, neg_distrib, abs_neg, neg_neg, lte_abs, abs_gte_zero, lte_lt_trans, lte_add_right, gt_zero_imp_pos, pos_gt_zero, sub_cancels, neg_lt_zero, neg_zero, lt_add_right, lt_add_pos, lt_add_one, neg_pos_is_neg, sub_moves_sides, add_neg_eq_zero, add_zero_right
from real.real_ring import mul_zero_left, mul_zero_right, lte_mul_nonneg_right, mul_neg_right, square_nonneg, mul_pos_pos, lte_mul_nonneg_left, mul_neg_left, real_mul_comm, mul_distrib_right, mul_distrib_left, mul_le_mul_nonneg
from real.real_seq import lt_imp_minus_pos, sub_zero_imp_eq
from real.mean_value import lte_imp_neg_lte_neg
from real.trig_identities_deep import add_sum_diff, sub_sum_diff, neg_diff
from real.exp import exp_add, exp_zero, exp_pos, exp_increasing, exp_unbounded, two, two_positive, two_nonzero, pow_suc, div_lt_div_pos
from real.log import exp_log_or_zero, exp_neg, log_some_of_pos_exists
from real.derivative_exp_log import exp_has_derivative_at
from real.derivative_basic import has_derivative_at, differentiable_at
from real.continuity_base import continuous
from order_set import closed_interval_set
from real.derivative_rules import derivative_pointwise_sub, derivative_pointwise_add
from real.derivative_quotient import pointwise_div_real, derivative_pointwise_div_const
from real.derivative_chain import derivative_compose
from real.derivative_affine_named import affine_real_has_derivative_at
from real.continuity_affine import affine_real
from real.calculus_api import is_derivative_fn, is_derivative_fn_iff
from real.calculus_quotient_continuity_examples import is_derivative_fn_imp_continuous
from real.intermediate_value import intermediate_value_closed_interval
from real.sqrt_inequalities import sqrt_value_mul_self, sqrt_value_nonneg
from data.basic.functions import compose, function_extensionality, function_eq_transport_predicate_rev
from data.basic.function_algebra import pointwise_neg, pointwise_add
from data.basic.witness import choose_or_default, choose_or_default_spec, exists_unique, exists_unique_intro
from algebra.field.field import mul_not_zero

numerals Real

/// The hyperbolic sine: (e^x - e^(-x)) / 2.
define sinh(x: Real) -> Real {
    (x.exp - (-x).exp) / two
}

/// The hyperbolic cosine: (e^x + e^(-x)) / 2.
define cosh(x: Real) -> Real {
    (x.exp + (-x).exp) / two
}

/// The hyperbolic tangent: sinh over cosh.
define tanh(x: Real) -> Real {
    sinh(x) / cosh(x)
}

/// The negation function on the reals.
define neg_fn(x: Real) -> Real {
    -x
}

/// The pointwise exponential of the negation: (-x).exp.
define exp_neg_fn(x: Real) -> Real {
    (-x).exp
}

/// The negation function is the affine map with slope minus one.
theorem neg_fn_eq_affine {
    neg_fn = affine_real(-Real.1, Real.0)
} by {
    forall(x: Real) {
        neg_fn(x) = -x
        affine_real(-Real.1, Real.0, x) = -Real.1 * x + Real.0
        -Real.1 * x = -x
        -Real.1 * x + Real.0 = -x
        neg_fn(x) = affine_real(-Real.1, Real.0, x)
    }
    function_extensionality(neg_fn, affine_real(-Real.1, Real.0))
    neg_fn = affine_real(-Real.1, Real.0)
}

/// The exponential of the negation is the composition of Real.exp with the negation.
theorem exp_neg_fn_eq_compose {
    exp_neg_fn = compose(Real.exp, neg_fn)
} by {
    forall(x: Real) {
        exp_neg_fn(x) = (-x).exp
        compose(Real.exp, neg_fn, x) = (neg_fn(x)).exp
        neg_fn(x) = -x
        (neg_fn(x)).exp = (-x).exp
        exp_neg_fn(x) = compose(Real.exp, neg_fn, x)
    }
    function_extensionality(exp_neg_fn, compose(Real.exp, neg_fn))
    exp_neg_fn = compose(Real.exp, neg_fn)
}

/// The hyperbolic sine is the pointwise quotient of the difference of Real.exp and
/// the negated exponential by the constant two.
theorem sinh_eq_pointwise {
    sinh = pointwise_div_real(
        pointwise_add(Real.exp, pointwise_neg(exp_neg_fn)),
        constant[Real, Real](two))
} by {
    forall(x: Real) {
        sinh(x) = (x.exp - (-x).exp) / two
        exp_neg_fn(x) = (-x).exp
        pointwise_neg(exp_neg_fn, x) = -exp_neg_fn(x)
        pointwise_neg(exp_neg_fn, x) = -(-x).exp
        pointwise_add(Real.exp, pointwise_neg(exp_neg_fn), x) =
            x.exp + pointwise_neg(exp_neg_fn, x)
        pointwise_add(Real.exp, pointwise_neg(exp_neg_fn), x) = x.exp + -(-x).exp
        x.exp + -(-x).exp = x.exp - (-x).exp
        pointwise_add(Real.exp, pointwise_neg(exp_neg_fn), x) = x.exp - (-x).exp
        pointwise_div_real(pointwise_add(Real.exp, pointwise_neg(exp_neg_fn)),
            constant[Real, Real](two), x) =
            pointwise_add(Real.exp, pointwise_neg(exp_neg_fn), x) / constant[Real, Real](two, x)
        constant[Real, Real](two, x) = two
        pointwise_div_real(pointwise_add(Real.exp, pointwise_neg(exp_neg_fn)),
            constant[Real, Real](two), x) = (x.exp - (-x).exp) / two
        sinh(x) = pointwise_div_real(pointwise_add(Real.exp, pointwise_neg(exp_neg_fn)),
            constant[Real, Real](two), x)
    }
    function_extensionality(sinh,
        pointwise_div_real(pointwise_add(Real.exp, pointwise_neg(exp_neg_fn)),
            constant[Real, Real](two)))
    sinh = pointwise_div_real(pointwise_add(Real.exp, pointwise_neg(exp_neg_fn)),
        constant[Real, Real](two))
}

/// The hyperbolic cosine is the pointwise quotient of the sum of Real.exp and the
/// negated exponential by the constant two.
theorem cosh_eq_pointwise {
    cosh = pointwise_div_real(pointwise_add(Real.exp, exp_neg_fn), constant[Real, Real](two))
} by {
    forall(x: Real) {
        cosh(x) = (x.exp + (-x).exp) / two
        exp_neg_fn(x) = (-x).exp
        pointwise_add(Real.exp, exp_neg_fn, x) = x.exp + exp_neg_fn(x)
        pointwise_add(Real.exp, exp_neg_fn, x) = x.exp + (-x).exp
        pointwise_div_real(pointwise_add(Real.exp, exp_neg_fn),
            constant[Real, Real](two), x) =
            pointwise_add(Real.exp, exp_neg_fn, x) / constant[Real, Real](two, x)
        constant[Real, Real](two, x) = two
        pointwise_div_real(pointwise_add(Real.exp, exp_neg_fn),
            constant[Real, Real](two), x) = (x.exp + (-x).exp) / two
        cosh(x) = pointwise_div_real(pointwise_add(Real.exp, exp_neg_fn),
            constant[Real, Real](two), x)
    }
    function_extensionality(cosh,
        pointwise_div_real(pointwise_add(Real.exp, exp_neg_fn), constant[Real, Real](two)))
    cosh = pointwise_div_real(pointwise_add(Real.exp, exp_neg_fn), constant[Real, Real](two))
}

// The derivatives.
//
// The negation has derivative -1 everywhere (affine), so by the chain rule
// (-x).exp has derivative -(-x).exp.  Adding the pointwise subtraction and
// division-by-constant rules to Real.exp' = Real.exp gives sinh' = cosh; the addition
// rule gives cosh' = sinh.

/// The negation function has derivative minus one at every point.
theorem neg_fn_has_derivative_at(x0: Real) {
    has_derivative_at(neg_fn, x0, -Real.1)
} by {
    affine_real_has_derivative_at(-Real.1, Real.0, x0)
    has_derivative_at(affine_real(-Real.1, Real.0), x0, -Real.1)
    define derivative_pred(h: Real -> Real) -> Bool {
        has_derivative_at(h, x0, -Real.1)
    }
    neg_fn_eq_affine
    neg_fn = affine_real(-Real.1, Real.0)
    derivative_pred(affine_real(-Real.1, Real.0))
    function_eq_transport_predicate_rev(derivative_pred, neg_fn, affine_real(-Real.1, Real.0))
    has_derivative_at(neg_fn, x0, -Real.1)
}

/// The exponential of the negation has derivative -(-x0).exp at x0.
theorem exp_neg_fn_has_derivative_at(x0: Real) {
    has_derivative_at(exp_neg_fn, x0, -(-x0).exp)
} by {
    neg_fn_has_derivative_at(x0)
    has_derivative_at(neg_fn, x0, -Real.1)
    exp_has_derivative_at(-x0)
    has_derivative_at(Real.exp, -x0, (-x0).exp)
    derivative_compose(neg_fn, Real.exp, x0, -Real.1, (-x0).exp)
    has_derivative_at(compose(Real.exp, neg_fn), x0, (-x0).exp * -Real.1)
    mul_neg_right((-x0).exp, Real.1)
    (-x0).exp * -Real.1 = -((-x0).exp * Real.1)
    (-x0).exp * Real.1 = (-x0).exp
    (-x0).exp * -Real.1 = -(-x0).exp
    has_derivative_at(compose(Real.exp, neg_fn), x0, -(-x0).exp)
    define derivative_pred(h: Real -> Real) -> Bool {
        has_derivative_at(h, x0, -(-x0).exp)
    }
    exp_neg_fn_eq_compose
    exp_neg_fn = compose(Real.exp, neg_fn)
    derivative_pred(compose(Real.exp, neg_fn))
    function_eq_transport_predicate_rev(derivative_pred, exp_neg_fn, compose(Real.exp, neg_fn))
    has_derivative_at(exp_neg_fn, x0, -(-x0).exp)
}

/// The hyperbolic sine has derivative cosh at every point.
theorem sinh_has_derivative_at(x0: Real) {
    has_derivative_at(sinh, x0, cosh(x0))
} by {
    exp_has_derivative_at(x0)
    has_derivative_at(Real.exp, x0, x0.exp)
    exp_neg_fn_has_derivative_at(x0)
    has_derivative_at(exp_neg_fn, x0, -(-x0).exp)
    derivative_pointwise_sub(Real.exp, exp_neg_fn, x0, x0.exp, -(-x0).exp)
    has_derivative_at(pointwise_add(Real.exp, pointwise_neg(exp_neg_fn)), x0,
        x0.exp + -(-(-x0).exp))
    neg_neg((-x0).exp)
    -(-(-x0).exp) = (-x0).exp
    x0.exp + -(-(-x0).exp) = x0.exp + (-x0).exp
    has_derivative_at(pointwise_add(Real.exp, pointwise_neg(exp_neg_fn)), x0,
        x0.exp + (-x0).exp)
    derivative_pointwise_div_const(
        pointwise_add(Real.exp, pointwise_neg(exp_neg_fn)), two, x0, x0.exp + (-x0).exp)
    has_derivative_at(
        pointwise_div_real(pointwise_add(Real.exp, pointwise_neg(exp_neg_fn)),
            constant[Real, Real](two)),
        x0,
        (x0.exp + (-x0).exp) / two)
    cosh(x0) = (x0.exp + (-x0).exp) / two
    (x0.exp + (-x0).exp) / two = cosh(x0)
    has_derivative_at(
        pointwise_div_real(pointwise_add(Real.exp, pointwise_neg(exp_neg_fn)),
            constant[Real, Real](two)),
        x0,
        cosh(x0))
    define derivative_pred(h: Real -> Real) -> Bool {
        has_derivative_at(h, x0, cosh(x0))
    }
    sinh_eq_pointwise
    sinh = pointwise_div_real(pointwise_add(Real.exp, pointwise_neg(exp_neg_fn)),
        constant[Real, Real](two))
    derivative_pred(pointwise_div_real(pointwise_add(Real.exp, pointwise_neg(exp_neg_fn)),
        constant[Real, Real](two)))
    function_eq_transport_predicate_rev(derivative_pred, sinh,
        pointwise_div_real(pointwise_add(Real.exp, pointwise_neg(exp_neg_fn)),
            constant[Real, Real](two)))
    has_derivative_at(sinh, x0, cosh(x0))
}

/// The hyperbolic cosine has derivative sinh at every point.
theorem cosh_has_derivative_at(x0: Real) {
    has_derivative_at(cosh, x0, sinh(x0))
} by {
    exp_has_derivative_at(x0)
    has_derivative_at(Real.exp, x0, x0.exp)
    exp_neg_fn_has_derivative_at(x0)
    has_derivative_at(exp_neg_fn, x0, -(-x0).exp)
    derivative_pointwise_add(Real.exp, exp_neg_fn, x0, x0.exp, -(-x0).exp)
    has_derivative_at(pointwise_add(Real.exp, exp_neg_fn), x0, x0.exp + -(-x0).exp)
    x0.exp + -(-x0).exp = x0.exp - (-x0).exp
    has_derivative_at(pointwise_add(Real.exp, exp_neg_fn), x0, x0.exp - (-x0).exp)
    derivative_pointwise_div_const(pointwise_add(Real.exp, exp_neg_fn), two, x0,
        x0.exp - (-x0).exp)
    has_derivative_at(
        pointwise_div_real(pointwise_add(Real.exp, exp_neg_fn), constant[Real, Real](two)),
        x0,
        (x0.exp - (-x0).exp) / two)
    sinh(x0) = (x0.exp - (-x0).exp) / two
    (x0.exp - (-x0).exp) / two = sinh(x0)
    has_derivative_at(
        pointwise_div_real(pointwise_add(Real.exp, exp_neg_fn), constant[Real, Real](two)),
        x0,
        sinh(x0))
    define derivative_pred(h: Real -> Real) -> Bool {
        has_derivative_at(h, x0, sinh(x0))
    }
    cosh_eq_pointwise
    cosh = pointwise_div_real(pointwise_add(Real.exp, exp_neg_fn), constant[Real, Real](two))
    derivative_pred(pointwise_div_real(pointwise_add(Real.exp, exp_neg_fn),
        constant[Real, Real](two)))
    function_eq_transport_predicate_rev(derivative_pred, cosh,
        pointwise_div_real(pointwise_add(Real.exp, exp_neg_fn), constant[Real, Real](two)))
    has_derivative_at(cosh, x0, sinh(x0))
}

/// The hyperbolic sine is everywhere differentiable with derivative cosh.
theorem sinh_is_derivative_fn {
    is_derivative_fn(sinh, cosh)
} by {
    forall(x: Real) {
        sinh_has_derivative_at(x)
        has_derivative_at(sinh, x, cosh(x))
    }
    is_derivative_fn_iff(sinh, cosh)
    is_derivative_fn(sinh, cosh)
}

/// The hyperbolic cosine is everywhere differentiable with derivative sinh.
theorem cosh_is_derivative_fn {
    is_derivative_fn(cosh, sinh)
} by {
    forall(x: Real) {
        cosh_has_derivative_at(x)
        has_derivative_at(cosh, x, sinh(x))
    }
    is_derivative_fn_iff(cosh, sinh)
    is_derivative_fn(cosh, sinh)
}

// The identity cosh^2 - sinh^2 = 1.
//
// Expanding the squares of (e^x + e^(-x)) and (e^x - e^(-x)) leaves
// 4 e^x e^(-x) / 4 = e^x e^(-x) = e^0 = 1.

/// Four as a real number.
let four = two * two

/// The square of a sum expands.
theorem sq_add_expand(a: Real, b: Real) {
    (a + b) * (a + b) = a * a + two * a * b + b * b
} by {
    mul_distrib_left(a, b, a + b)
    (a + b) * (a + b) = a * (a + b) + b * (a + b)
    mul_distrib_right(a, a, b)
    a * (a + b) = a * a + a * b
    mul_distrib_right(b, a, b)
    b * (a + b) = b * a + b * b
    real_mul_comm(b, a)
    b * a = a * b
    b * (a + b) = a * b + b * b
    a * (a + b) + b * (a + b) = (a * a + a * b) + (a * b + b * b)
    (a * a + a * b) + (a * b + b * b) = a * a + (a * b + a * b) + b * b
    a * b + a * b = two * a * b
    a * a + (a * b + a * b) + b * b = a * a + two * a * b + b * b
    (a + b) * (a + b) = a * a + two * a * b + b * b
}

/// The square of a difference expands.
theorem sq_sub_expand(a: Real, b: Real) {
    (a - b) * (a - b) = a * a - two * a * b + b * b
} by {
    a - b = a + -b
    (a - b) * (a - b) = (a + -b) * (a + -b)
    sq_add_expand(a, -b)
    (a + -b) * (a + -b) = a * a + two * a * -b + (-b) * (-b)
    mul_neg_right(a, b)
    a * -b = -(a * b)
    two * (a * -b) = two * -(a * b)
    two * -(a * b) = -(two * (a * b))
    a * a + two * (a * -b) + (-b) * (-b) = a * a - two * (a * b) + (-b) * (-b)
    mul_neg_left(b, -b)
    (-b) * (-b) = -(b * (-b))
    mul_neg_right(b, b)
    b * (-b) = -(b * b)
    -(-(b * b)) = b * b
    (-b) * (-b) = b * b
    a * a - two * (a * b) + b * b = a * a - two * a * b + b * b
    (a - b) * (a - b) = a * a - two * a * b + b * b
}

/// The difference of two squares factors.
theorem sq_diff_factor(a: Real, b: Real) {
    a * a - b * b = (a - b) * (a + b)
} by {
    (a - b) * (a + b) = a * (a + b) - b * (a + b)
    a * (a + b) = a * a + a * b
    b * (a + b) = b * a + b * b
    real_mul_comm(b, a)
    b * a = a * b
    b * (a + b) = a * b + b * b
    a * (a + b) - b * (a + b) = (a * a + a * b) - (a * b + b * b)
    (a * a + a * b) - (a * b + b * b) = a * a - b * b
    (a - b) * (a + b) = a * a - b * b
    a * a - b * b = (a - b) * (a + b)
}

/// The difference of the squares of a sum and a difference is four times the product.
theorem sq_sum_sub_sq_diff(a: Real, b: Real) {
    (a + b) * (a + b) - (a - b) * (a - b) = four * a * b
} by {
    // Difference of squares: u^2 - v^2 = (u - v)(u + v).
    sq_diff_factor(a + b, a - b)
    (a + b) * (a + b) - (a - b) * (a - b) =
        ((a + b) - (a - b)) * ((a + b) + (a - b))
    sub_sum_diff(a, b)
    (a + b) - (a - b) = two * b
    add_sum_diff(a, b)
    (a + b) + (a - b) = two * a
    ((a + b) - (a - b)) * ((a + b) + (a - b)) = (two * b) * (two * a)
    (two * b) * (two * a) = (two * two) * (b * a)
    real_mul_comm(b, a)
    b * a = a * b
    (two * two) * (b * a) = (two * two) * (a * b)
    four = two * two
    (two * two) * (a * b) = four * (a * b)
    four * (a * b) = four * a * b
    (a + b) * (a + b) - (a - b) * (a - b) = four * a * b
}

// Cross-product identities for the addition formulas.
//
// Expanding the left-hand sides with the distributivity laws and collecting
// the cross terms leaves twice the outer products, exactly as in the sine and
// cosine addition formulas of trig_identities.ac.

/// The sum of the cross products of the differences and sums is twice the difference of the outer products.
theorem cross_sum_diff(a: Real, b: Real, c: Real, d: Real) {
    (a - b) * (c + d) + (a + b) * (c - d) = two * a * c - two * b * d
} by {
    // (a - b)(c + d) = a(c + d) - b(c + d)
    mul_distrib_right(a, c, d)
    a * (c + d) = a * c + a * d
    mul_distrib_right(b, c, d)
    b * (c + d) = b * c + b * d
    (a - b) * (c + d) = a * (c + d) - b * (c + d)
    (a - b) * (c + d) = (a * c + a * d) - (b * c + b * d)
    // (a + b)(c - d) = a(c - d) + b(c - d) = (a * c - a * d) + (b * c - b * d)
    mul_distrib_right(a, c, -d)
    a * (c - d) = a * c + a * -d
    mul_neg_right(a, d)
    a * -d = -(a * d)
    a * (c - d) = a * c - a * d
    mul_distrib_right(b, c, -d)
    b * (c - d) = b * c + b * -d
    mul_neg_right(b, d)
    b * -d = -(b * d)
    b * (c - d) = b * c - b * d
    (a + b) * (c - d) = a * (c - d) + b * (c - d)
    (a + b) * (c - d) = (a * c - a * d) + (b * c - b * d)
    (a - b) * (c + d) + (a + b) * (c - d) =
        ((a * c + a * d) - (b * c + b * d)) + ((a * c - a * d) + (b * c - b * d))
    add_comm((a * c + a * d) - (b * c + b * d), (a * c - a * d) + (b * c - b * d))
    ((a * c + a * d) - (b * c + b * d)) + ((a * c - a * d) + (b * c - b * d)) =
        ((a * c - a * d) + (b * c - b * d)) + ((a * c + a * d) - (b * c + b * d))
    add_assoc(a * c - a * d, b * c - b * d, (a * c + a * d) - (b * c + b * d))
    ((a * c - a * d) + (b * c - b * d)) + ((a * c + a * d) - (b * c + b * d)) =
        (a * c - a * d) + ((b * c - b * d) + ((a * c + a * d) - (b * c + b * d)))
    add_comm(b * c - b * d, (a * c + a * d) - (b * c + b * d))
    (b * c - b * d) + ((a * c + a * d) - (b * c + b * d)) =
        ((a * c + a * d) - (b * c + b * d)) + (b * c - b * d)
    (a * c - a * d) + ((b * c - b * d) + ((a * c + a * d) - (b * c + b * d))) =
        (a * c - a * d) + (((a * c + a * d) - (b * c + b * d)) + (b * c - b * d))
    add_assoc(a * c - a * d, (a * c + a * d) - (b * c + b * d), b * c - b * d)
    ((a * c - a * d) + ((a * c + a * d) - (b * c + b * d))) + (b * c - b * d) =
        (a * c - a * d) + (((a * c + a * d) - (b * c + b * d)) + (b * c - b * d))
    (a * c - a * d) + ((b * c - b * d) + ((a * c + a * d) - (b * c + b * d))) =
        ((a * c - a * d) + ((a * c + a * d) - (b * c + b * d))) + (b * c - b * d)
    sub_sum_diff(a * c, a * d)
    (a * c + a * d) - (a * c - a * d) = two * a * d
    add_sum_diff(a * c, a * d)
    (a * c + a * d) + (a * c - a * d) = two * a * c
    (a * c - a * d) + ((a * c + a * d) - (b * c + b * d)) =
        ((a * c - a * d) + (a * c + a * d)) - (b * c + b * d)
    add_comm(a * c - a * d, a * c + a * d)
    (a * c - a * d) + (a * c + a * d) = (a * c + a * d) + (a * c - a * d)
    (a * c - a * d) + (a * c + a * d) = two * a * c
    (a * c - a * d) + ((a * c + a * d) - (b * c + b * d)) =
        two * a * c - (b * c + b * d)
    (a * c - a * d) + ((b * c - b * d) + ((a * c + a * d) - (b * c + b * d))) =
        two * a * c - (b * c + b * d) + (b * c - b * d)
    (a - b) * (c + d) + (a + b) * (c - d) =
        two * a * c - (b * c + b * d) + (b * c - b * d)
    add_assoc(two * a * c - (b * c + b * d), b * c, -b * d)
    (two * a * c - (b * c + b * d)) + (b * c + -b * d) =
        (two * a * c - (b * c + b * d) + b * c) + -b * d
    two * a * c - (b * c + b * d) + (b * c - b * d) =
        two * a * c - (b * c + b * d) + b * c + -b * d
    two * a * c - (b * c + b * d) + (b * c - b * d) =
        two * a * c - (b * c + b * d) + b * c - b * d
    (b * c + b * d) - (b * c - b * d) = two * b * d
    sub_sum_diff(b * c, b * d)
    (b * c + b * d) - (b * c - b * d) = two * b * d
    two * a * c - (b * c + b * d) + (b * c - b * d) = two * a * c - two * b * d
    (a - b) * (c + d) + (a + b) * (c - d) = two * a * c - two * b * d
}

/// The sum of the cross products of the sums and differences is twice the sum of the outer products.
theorem cross_sum_add(a: Real, b: Real, c: Real, d: Real) {
    (a + b) * (c + d) + (a - b) * (c - d) = two * a * c + two * b * d
} by {
    // (a + b)(c + d) = (a * c + a * d) + (b * c + b * d)
    mul_distrib_right(a, c, d)
    a * (c + d) = a * c + a * d
    mul_distrib_right(b, c, d)
    b * (c + d) = b * c + b * d
    (a + b) * (c + d) = a * (c + d) + b * (c + d)
    (a + b) * (c + d) = (a * c + a * d) + (b * c + b * d)
    // (a - b)(c - d) = (a * c - a * d) - (b * c - b * d)
    mul_distrib_right(a, c, -d)
    a * (c - d) = a * c + a * -d
    mul_neg_right(a, d)
    a * -d = -(a * d)
    a * (c - d) = a * c - a * d
    mul_distrib_right(b, c, -d)
    b * (c - d) = b * c + b * -d
    mul_neg_right(b, d)
    b * -d = -(b * d)
    b * (c - d) = b * c - b * d
    (a - b) * (c - d) = a * (c - d) - b * (c - d)
    (a - b) * (c - d) = (a * c - a * d) - (b * c - b * d)
    (a + b) * (c + d) + (a - b) * (c - d) =
        ((a * c + a * d) + (b * c + b * d)) + ((a * c - a * d) - (b * c - b * d))
    add_comm((a * c + a * d) + (b * c + b * d), (a * c - a * d) - (b * c - b * d))
    ((a * c + a * d) + (b * c + b * d)) + ((a * c - a * d) - (b * c - b * d)) =
        ((a * c - a * d) - (b * c - b * d)) + ((a * c + a * d) + (b * c + b * d))
    add_assoc(a * c - a * d, -(b * c - b * d), (a * c + a * d) + (b * c + b * d))
    ((a * c - a * d) - (b * c - b * d)) + ((a * c + a * d) + (b * c + b * d)) =
        (a * c - a * d) + (-(b * c - b * d) + ((a * c + a * d) + (b * c + b * d)))
    add_comm(-(b * c - b * d), (a * c + a * d) + (b * c + b * d))
    -(b * c - b * d) + ((a * c + a * d) + (b * c + b * d)) =
        ((a * c + a * d) + (b * c + b * d)) + (-(b * c - b * d))
    (a * c - a * d) + (-(b * c - b * d) + ((a * c + a * d) + (b * c + b * d))) =
        (a * c - a * d) + (((a * c + a * d) + (b * c + b * d)) + (-(b * c - b * d)))
    add_assoc(a * c - a * d, (a * c + a * d) + (b * c + b * d), -(b * c - b * d))
    ((a * c - a * d) + ((a * c + a * d) + (b * c + b * d))) + (-(b * c - b * d)) =
        (a * c - a * d) + (((a * c + a * d) + (b * c + b * d)) + (-(b * c - b * d)))
    (a * c - a * d) + (-(b * c - b * d) + ((a * c + a * d) + (b * c + b * d))) =
        ((a * c - a * d) + ((a * c + a * d) + (b * c + b * d))) + (-(b * c - b * d))
    add_assoc(a * c - a * d, a * c + a * d, b * c + b * d)
    ((a * c - a * d) + (a * c + a * d)) + (b * c + b * d) =
        (a * c - a * d) + ((a * c + a * d) + (b * c + b * d))
    (a * c - a * d) + (-(b * c - b * d) + ((a * c + a * d) + (b * c + b * d))) =
        ((a * c - a * d) + (a * c + a * d)) + (b * c + b * d) + (-(b * c - b * d))
    add_sum_diff(a * c, a * d)
    (a * c + a * d) + (a * c - a * d) = two * a * c
    add_comm(a * c + a * d, a * c - a * d)
    (a * c + a * d) + (a * c - a * d) = (a * c - a * d) + (a * c + a * d)
    (a * c - a * d) + (a * c + a * d) = two * a * c
    (a * c - a * d) + (-(b * c - b * d) + ((a * c + a * d) + (b * c + b * d))) =
        two * a * c + (b * c + b * d) + (-(b * c - b * d))
    (a + b) * (c + d) + (a - b) * (c - d) =
        two * a * c + (b * c + b * d) - (b * c - b * d)
    add_assoc(two * a * c, b * c + b * d, -(b * c - b * d))
    (two * a * c + (b * c + b * d)) + (-(b * c - b * d)) =
        two * a * c + ((b * c + b * d) + (-(b * c - b * d)))
    (two * a * c + (b * c + b * d)) - (b * c - b * d) =
        (two * a * c + (b * c + b * d)) + (-(b * c - b * d))
    two * a * c + (b * c + b * d) - (b * c - b * d) =
        two * a * c + ((b * c + b * d) - (b * c - b * d))
    sub_sum_diff(b * c, b * d)
    (b * c + b * d) - (b * c - b * d) = two * b * d
    two * a * c + ((b * c + b * d) - (b * c - b * d)) = two * a * c + two * b * d
    (a + b) * (c + d) + (a - b) * (c - d) = two * a * c + two * b * d
}

/// Cancelling a common factor of two in a quotient of a difference.
theorem two_div_sub(u: Real, v: Real) {
    (two * u - two * v) / (two * two) = (u - v) / two
} by {
    two * u - two * v = two * (u - v)
    two_nonzero
    two != Real.0
    mul_not_zero(two, two)
    two * two != Real.0
    div_cancel_common(u - v, two, two)
    ((u - v) * two) / (two * two) = (u - v) / two
    real_mul_comm(u - v, two)
    (u - v) * two = two * (u - v)
    (two * (u - v)) / (two * two) = (u - v) / two
    two * (u - v) = two * u - two * v
    (two * u - two * v) / (two * two) = (u - v) / two
}

/// Cancelling a common factor of two in a quotient of a sum.
theorem two_div_add(u: Real, v: Real) {
    (two * u + two * v) / (two * two) = (u + v) / two
} by {
    two * u + two * v = two * (u + v)
    two_nonzero
    two != Real.0
    mul_not_zero(two, two)
    two * two != Real.0
    div_cancel_common(u + v, two, two)
    ((u + v) * two) / (two * two) = (u + v) / two
    real_mul_comm(u + v, two)
    (u + v) * two = two * (u + v)
    (two * (u + v)) / (two * two) = (u + v) / two
    two * (u + v) = two * u + two * v
    (two * u + two * v) / (two * two) = (u + v) / two
}

/// The product of a sum and a mirrored difference is the difference of the squares.
theorem mul_add_sub_sq(a: Real, b: Real) {
    (a + b) * (b - a) = b * b - a * a
} by {
    sq_diff_factor(b, a)
    b * b - a * a = (b - a) * (b + a)
    real_mul_comm(b - a, b + a)
    (b - a) * (b + a) = (b + a) * (b - a)
    b + a = a + b
    (b + a) * (b - a) = (a + b) * (b - a)
    b * b - a * a = (a + b) * (b - a)
    (a + b) * (b - a) = b * b - a * a
}

/// A sum of three terms with a cancellation.
theorem add_cancel_mid(a: Real, b: Real, c: Real) {
    (a + b) + -b = a
} by {
    add_assoc(a, b, -b)
    (a + b) + -b = a + (b + -b)
    add_neg_eq_zero(b)
    b + -b = Real.0
    a + (b + -b) = a + Real.0
    add_zero_right(a)
    a + Real.0 = a
    (a + b) + -b = a
}

/// Swapping and cancelling: (a + b) + -a = b.
theorem add_swap_cancel(a: Real, b: Real) {
    (a + b) + -a = b
} by {
    add_comm(a, b)
    a + b = b + a
    (a + b) + -a = (b + a) + -a
    add_cancel_mid(b, a, b)
    (b + a) + -a = b
    (a + b) + -a = b
}

/// The sum of two differences sharing a term: (a - b) + (c - a) = c - b.
theorem add_sub_swap(a: Real, b: Real, c: Real) {
    (a - b) + (c - a) = c - b
} by {
    add_comm(a - b, c - a)
    (a - b) + (c - a) = (c - a) + (a - b)
    add_assoc(c, -a, a - b)
    (c + -a) + (a - b) = c + (-a + (a - b))
    c - a = c + -a
    (c - a) + (a - b) = (c + -a) + (a - b)
    (c - a) + (a - b) = c + (-a + (a - b))
    a - b = a + -b
    -a + (a - b) = -a + (a + -b)
    add_assoc(-a, a, -b)
    (-a + a) + -b = -a + (a + -b)
    add_comm(-a, a)
    -a + a = a + -a
    add_neg_eq_zero(a)
    a + -a = Real.0
    -a + a = Real.0
    (-a + a) + -b = Real.0 + -b
    Real.0 + -b = -b
    -a + (a + -b) = -b
    -a + (a - b) = -b
    c + (-a + (a - b)) = c + -b
    c + -b = c - b
    (c - a) + (a - b) = c - b
    (a - b) + (c - a) = c - b
}

/// The sum of a term and the negation of a difference is the difference of the
/// terms: x + s - (s - x) = 2x.
theorem add_sub_neg_diff(x: Real, s: Real) {
    (x + s) - (s - x) = two * x
} by {
    sub_sum_diff(s, x)
    (s + x) - (s - x) = two * x
    add_comm(s, x)
    s + x = x + s
    (x + s) - (s - x) = two * x
}

/// The difference of the squared quotients is the quotient of the square difference.
theorem cosh_sq_sub_sinh_sq_exp(x: Real) {
    ((x.exp + (-x).exp) / two) * ((x.exp + (-x).exp) / two) -
        ((x.exp - (-x).exp) / two) * ((x.exp - (-x).exp) / two) =
        (four * x.exp * (-x).exp) / (two * two)
} by {
    two_nonzero
    two != Real.0
    mul_div(x.exp + (-x).exp, two, x.exp + (-x).exp, two)
    ((x.exp + (-x).exp) / two) * ((x.exp + (-x).exp) / two) =
        ((x.exp + (-x).exp) * (x.exp + (-x).exp)) / (two * two)
    mul_div(x.exp - (-x).exp, two, x.exp - (-x).exp, two)
    ((x.exp - (-x).exp) / two) * ((x.exp - (-x).exp) / two) =
        ((x.exp - (-x).exp) * (x.exp - (-x).exp)) / (two * two)
    sq_sum_sub_sq_diff(x.exp, (-x).exp)
    (x.exp + (-x).exp) * (x.exp + (-x).exp) - (x.exp - (-x).exp) * (x.exp - (-x).exp) =
        four * x.exp * (-x).exp
    ((x.exp + (-x).exp) * (x.exp + (-x).exp)) / (two * two) -
        ((x.exp - (-x).exp) * (x.exp - (-x).exp)) / (two * two) =
        ((x.exp + (-x).exp) * (x.exp + (-x).exp) -
         (x.exp - (-x).exp) * (x.exp - (-x).exp)) / (two * two)
    ((x.exp + (-x).exp) * (x.exp + (-x).exp) -
     (x.exp - (-x).exp) * (x.exp - (-x).exp)) / (two * two) =
        (four * x.exp * (-x).exp) / (two * two)
}

/// The difference of the squares of the hyperbolic cosine and sine is one.
theorem cosh_sq_sub_sinh_sq(x: Real) {
    cosh(x) * cosh(x) - sinh(x) * sinh(x) = Real.1
} by {
    cosh(x) = (x.exp + (-x).exp) / two
    sinh(x) = (x.exp - (-x).exp) / two
    cosh(x) * cosh(x) = ((x.exp + (-x).exp) / two) * ((x.exp + (-x).exp) / two)
    sinh(x) * sinh(x) = ((x.exp - (-x).exp) / two) * ((x.exp - (-x).exp) / two)
    cosh_sq_sub_sinh_sq_exp(x)
    cosh(x) * cosh(x) - sinh(x) * sinh(x) = (four * x.exp * (-x).exp) / (two * two)
    exp_add(x, -x)
    (x + -x).exp = x.exp * (-x).exp
    x + -x = Real.0
    (Real.0).exp = x.exp * (-x).exp
    exp_zero
    (Real.0).exp = Real.1
    x.exp * (-x).exp = Real.1
    four * x.exp * (-x).exp = four * (x.exp * (-x).exp)
    four * (x.exp * (-x).exp) = four * Real.1
    four * Real.1 = four
    (four * x.exp * (-x).exp) / (two * two) = four / (two * two)
    four = two * two
    four / (two * two) = four / four
    two_nonzero
    two != Real.0
    mul_not_zero(two, two)
    two * two != Real.0
    four != Real.0
    mul_div_cancel(four, Real.1)
    (four * Real.1) / four = Real.1
    four * Real.1 = four
    four / four = Real.1
    four / (two * two) = Real.1
    (four * x.exp * (-x).exp) / (two * two) = Real.1
    cosh(x) * cosh(x) - sinh(x) * sinh(x) = Real.1
}

// The addition formulas.
//
// Both sides expand with exp_add(x, y) and exp_add(-x, -y) into the same
// combination of e^x e^y and e^(-x) e^(-y).

/// The hyperbolic sine of a sum is the sum of the cross products.
theorem sinh_add(x: Real, y: Real) {
    sinh(x + y) = sinh(x) * cosh(y) + cosh(x) * sinh(y)
} by {
    exp_add(x, y)
    (x + y).exp = x.exp * y.exp
    exp_add(-x, -y)
    (-x + -y).exp = (-x).exp * (-y).exp
    -(x + y) = -x + -y
    (-(x + y)).exp = (-x).exp * (-y).exp
    sinh(x + y) = ((x + y).exp - (-(x + y)).exp) / two
    (x + y).exp - (-(x + y)).exp = x.exp * y.exp - (-x).exp * (-y).exp
    sinh(x + y) = (x.exp * y.exp - (-x).exp * (-y).exp) / two
    sinh(x) = (x.exp - (-x).exp) / two
    cosh(y) = (y.exp + (-y).exp) / two
    cosh(x) = (x.exp + (-x).exp) / two
    sinh(y) = (y.exp - (-y).exp) / two
    sinh(x) * cosh(y) = ((x.exp - (-x).exp) / two) * ((y.exp + (-y).exp) / two)
    cosh(x) * sinh(y) = ((x.exp + (-x).exp) / two) * ((y.exp - (-y).exp) / two)
    two_nonzero
    two != Real.0
    mul_div(x.exp - (-x).exp, two, y.exp + (-y).exp, two)
    ((x.exp - (-x).exp) / two) * ((y.exp + (-y).exp) / two) =
        ((x.exp - (-x).exp) * (y.exp + (-y).exp)) / (two * two)
    mul_div(x.exp + (-x).exp, two, y.exp - (-y).exp, two)
    ((x.exp + (-x).exp) / two) * ((y.exp - (-y).exp) / two) =
        ((x.exp + (-x).exp) * (y.exp - (-y).exp)) / (two * two)
    sinh(x) * cosh(y) + cosh(x) * sinh(y) =
        ((x.exp - (-x).exp) * (y.exp + (-y).exp)) / (two * two) +
        ((x.exp + (-x).exp) * (y.exp - (-y).exp)) / (two * two)
    ((x.exp - (-x).exp) * (y.exp + (-y).exp)) / (two * two) +
        ((x.exp + (-x).exp) * (y.exp - (-y).exp)) / (two * two) =
        ((x.exp - (-x).exp) * (y.exp + (-y).exp) +
         (x.exp + (-x).exp) * (y.exp - (-y).exp)) / (two * two)
    cross_sum_diff(x.exp, (-x).exp, y.exp, (-y).exp)
    (x.exp - (-x).exp) * (y.exp + (-y).exp) +
        (x.exp + (-x).exp) * (y.exp - (-y).exp) =
        two * x.exp * y.exp - two * (-x).exp * (-y).exp
    ((x.exp - (-x).exp) * (y.exp + (-y).exp) +
     (x.exp + (-x).exp) * (y.exp - (-y).exp)) / (two * two) =
        (two * x.exp * y.exp - two * (-x).exp * (-y).exp) / (two * two)
    sinh(x) * cosh(y) + cosh(x) * sinh(y) =
        (two * x.exp * y.exp - two * (-x).exp * (-y).exp) / (two * two)
    two_div_sub(x.exp * y.exp, (-x).exp * (-y).exp)
    (two * (x.exp * y.exp) - two * ((-x).exp * (-y).exp)) / (two * two) =
        (x.exp * y.exp - (-x).exp * (-y).exp) / two
    (two * x.exp * y.exp - two * (-x).exp * (-y).exp) / (two * two) =
        (x.exp * y.exp - (-x).exp * (-y).exp) / two
    sinh(x) * cosh(y) + cosh(x) * sinh(y) =
        (x.exp * y.exp - (-x).exp * (-y).exp) / two
    sinh(x + y) = sinh(x) * cosh(y) + cosh(x) * sinh(y)
}

/// The hyperbolic cosine of a sum is the sum of the like products.
theorem cosh_add(x: Real, y: Real) {
    cosh(x + y) = cosh(x) * cosh(y) + sinh(x) * sinh(y)
} by {
    exp_add(x, y)
    (x + y).exp = x.exp * y.exp
    exp_add(-x, -y)
    (-x + -y).exp = (-x).exp * (-y).exp
    -(x + y) = -x + -y
    (-(x + y)).exp = (-x).exp * (-y).exp
    cosh(x + y) = ((x + y).exp + (-(x + y)).exp) / two
    (x + y).exp + (-(x + y)).exp = x.exp * y.exp + (-x).exp * (-y).exp
    cosh(x + y) = (x.exp * y.exp + (-x).exp * (-y).exp) / two
    cosh(x) = (x.exp + (-x).exp) / two
    cosh(y) = (y.exp + (-y).exp) / two
    sinh(x) = (x.exp - (-x).exp) / two
    sinh(y) = (y.exp - (-y).exp) / two
    cosh(x) * cosh(y) = ((x.exp + (-x).exp) / two) * ((y.exp + (-y).exp) / two)
    sinh(x) * sinh(y) = ((x.exp - (-x).exp) / two) * ((y.exp - (-y).exp) / two)
    two_nonzero
    two != Real.0
    mul_div(x.exp + (-x).exp, two, y.exp + (-y).exp, two)
    ((x.exp + (-x).exp) / two) * ((y.exp + (-y).exp) / two) =
        ((x.exp + (-x).exp) * (y.exp + (-y).exp)) / (two * two)
    mul_div(x.exp - (-x).exp, two, y.exp - (-y).exp, two)
    ((x.exp - (-x).exp) / two) * ((y.exp - (-y).exp) / two) =
        ((x.exp - (-x).exp) * (y.exp - (-y).exp)) / (two * two)
    cosh(x) * cosh(y) + sinh(x) * sinh(y) =
        ((x.exp + (-x).exp) * (y.exp + (-y).exp)) / (two * two) +
        ((x.exp - (-x).exp) * (y.exp - (-y).exp)) / (two * two)
    ((x.exp + (-x).exp) * (y.exp + (-y).exp)) / (two * two) +
        ((x.exp - (-x).exp) * (y.exp - (-y).exp)) / (two * two) =
        ((x.exp + (-x).exp) * (y.exp + (-y).exp) +
         (x.exp - (-x).exp) * (y.exp - (-y).exp)) / (two * two)
    cross_sum_add(x.exp, (-x).exp, y.exp, (-y).exp)
    (x.exp + (-x).exp) * (y.exp + (-y).exp) +
        (x.exp - (-x).exp) * (y.exp - (-y).exp) =
        two * x.exp * y.exp + two * (-x).exp * (-y).exp
    ((x.exp + (-x).exp) * (y.exp + (-y).exp) +
     (x.exp - (-x).exp) * (y.exp - (-y).exp)) / (two * two) =
        (two * x.exp * y.exp + two * (-x).exp * (-y).exp) / (two * two)
    cosh(x) * cosh(y) + sinh(x) * sinh(y) =
        (two * x.exp * y.exp + two * (-x).exp * (-y).exp) / (two * two)
    two_div_add(x.exp * y.exp, (-x).exp * (-y).exp)
    (two * (x.exp * y.exp) + two * ((-x).exp * (-y).exp)) / (two * two) =
        (x.exp * y.exp + (-x).exp * (-y).exp) / two
    (two * x.exp * y.exp + two * (-x).exp * (-y).exp) / (two * two) =
        (x.exp * y.exp + (-x).exp * (-y).exp) / two
    cosh(x) * cosh(y) + sinh(x) * sinh(y) =
        (x.exp * y.exp + (-x).exp * (-y).exp) / two
    cosh(x + y) = cosh(x) * cosh(y) + sinh(x) * sinh(y)
}

// Parity.

/// The hyperbolic sine is odd.
theorem sinh_neg(x: Real) {
    sinh(-x) = -sinh(x)
} by {
    sinh(-x) = ((-x).exp - (-(-x)).exp) / two
    neg_neg(x)
    -(-x) = x
    (-(-x)).exp = x.exp
    (-x).exp - (-(-x)).exp = (-x).exp - x.exp
    sinh(-x) = ((-x).exp - x.exp) / two
    sinh(x) = (x.exp - (-x).exp) / two
    -sinh(x) = -(x.exp - (-x).exp) / two
    -(x.exp - (-x).exp) / two = (-(x.exp - (-x).exp)) / two
    -(x.exp - (-x).exp) = (-x).exp - x.exp
    -sinh(x) = ((-x).exp - x.exp) / two
    sinh(-x) = -sinh(x)
}

/// The hyperbolic cosine is even.
theorem cosh_neg(x: Real) {
    cosh(-x) = cosh(x)
} by {
    cosh(-x) = ((-x).exp + (-(-x)).exp) / two
    neg_neg(x)
    -(-x) = x
    (-(-x)).exp = x.exp
    (-x).exp + (-(-x)).exp = (-x).exp + x.exp
    cosh(-x) = ((-x).exp + x.exp) / two
    cosh(x) = (x.exp + (-x).exp) / two
    (-x).exp + x.exp = x.exp + (-x).exp
    ((-x).exp + x.exp) / two = (x.exp + (-x).exp) / two
    cosh(-x) = cosh(x)
}

// Positivity and strict monotonicity of sinh.
//
// For t > 0, sinh(t) = (e^t - e^(-t))/2 > 0 because Real.exp is strictly
// increasing and t > -t.  The hyperbolic cosine is positive everywhere as the
// average of two positive numbers.  Then the difference formula
//     sinh(y) - sinh(x) = 2 cosh((x + y)/2) sinh((y - x)/2),
// which follows from the addition formula and parity, shows sinh is strictly
// increasing.

/// The hyperbolic sine is positive on positive reals.
theorem sinh_pos_of_pos(x: Real) {
    x > Real.0 implies sinh(x) > Real.0
} by {
    if x > Real.0 {
        neg_pos_is_neg(x)
        -x < Real.0
        lt_trans(-x, Real.0, x)
        -x < x
        exp_increasing(-x, x)
        (-x).exp < x.exp
        lt_imp_minus_pos((-x).exp, x.exp)
        (x.exp - (-x).exp).is_positive
        pos_gt_zero(x.exp - (-x).exp)
        x.exp - (-x).exp > Real.0
        two_positive
        two > Real.0
        div_lt_div_pos(Real.0, x.exp - (-x).exp, two)
        Real.0 / two < (x.exp - (-x).exp) / two
        Real.0 / two = Real.0
        Real.0 < (x.exp - (-x).exp) / two
        sinh(x) = (x.exp - (-x).exp) / two
        sinh(x) > Real.0
    }
}

/// The hyperbolic cosine is positive everywhere.
theorem cosh_pos(x: Real) {
    cosh(x) > Real.0
} by {
    exp_pos(x)
    x.exp > Real.0
    exp_pos(-x)
    (-x).exp > Real.0
    lt_add_right(Real.0, x.exp, (-x).exp)
    Real.0 + (-x).exp < x.exp + (-x).exp
    Real.0 + (-x).exp = (-x).exp
    (-x).exp < x.exp + (-x).exp
    lt_trans(Real.0, (-x).exp, x.exp + (-x).exp)
    Real.0 < x.exp + (-x).exp
    two_positive
    two > Real.0
    div_lt_div_pos(Real.0, x.exp + (-x).exp, two)
    Real.0 / two < (x.exp + (-x).exp) / two
    Real.0 / two = Real.0
    Real.0 < (x.exp + (-x).exp) / two
    cosh(x) = (x.exp + (-x).exp) / two
    cosh(x) > Real.0
}

/// The hyperbolic sine of a difference in terms of the products.
theorem sinh_sub(x: Real, y: Real) {
    sinh(x - y) = sinh(x) * cosh(y) - cosh(x) * sinh(y)
} by {
    sinh_add(x, -y)
    sinh(x + -y) = sinh(x) * cosh(-y) + cosh(x) * sinh(-y)
    cosh_neg(y)
    cosh(-y) = cosh(y)
    sinh_neg(y)
    sinh(-y) = -sinh(y)
    sinh(x) * cosh(-y) + cosh(x) * sinh(-y) = sinh(x) * cosh(y) + cosh(x) * -sinh(y)
    cosh(x) * -sinh(y) = -(cosh(x) * sinh(y))
    sinh(x) * cosh(y) + cosh(x) * -sinh(y) = sinh(x) * cosh(y) - cosh(x) * sinh(y)
    sinh(x + -y) = sinh(x) * cosh(y) - cosh(x) * sinh(y)
    x + -y = x - y
    sinh(x - y) = sinh(x) * cosh(y) - cosh(x) * sinh(y)
}

/// Half the sum plus half the difference is the larger point.
theorem half_sum_add_half_diff_local(x: Real, y: Real) {
    (x + y) / two + (y - x) / two = y
} by {
    add_sum_diff(y, x)
    (y + x) + (y - x) = two * y
    y + x = x + y
    (x + y) + (y - x) = two * y
    (x + y) / two + (y - x) / two = ((x + y) + (y - x)) / two
    ((x + y) + (y - x)) / two = (two * y) / two
    two_nonzero
    two != Real.0
    mul_div_cancel(y, two)
    (two * y) / two = y
    (x + y) / two + (y - x) / two = y
}

/// Half the sum minus half the difference is the smaller point.
theorem half_sum_sub_half_diff_local(x: Real, y: Real) {
    (x + y) / two - (y - x) / two = x
} by {
    sub_sum_diff(y, x)
    (y + x) - (y - x) = two * x
    y + x = x + y
    (x + y) - (y - x) = two * x
    (x + y) / two - (y - x) / two = ((x + y) - (y - x)) / two
    ((x + y) - (y - x)) / two = (two * x) / two
    two_nonzero
    two != Real.0
    mul_div_cancel(x, two)
    (two * x) / two = x
    (x + y) / two - (y - x) / two = x
}

/// The difference of the hyperbolic sines as a product.
theorem sinh_sub_formula(x: Real, y: Real) {
    sinh(y) - sinh(x) = two * cosh((x + y) / two) * sinh((y - x) / two)
} by {
    sinh_add((x + y) / two, (y - x) / two)
    sinh((x + y) / two + (y - x) / two) =
        sinh((x + y) / two) * cosh((y - x) / two) + cosh((x + y) / two) * sinh((y - x) / two)
    half_sum_add_half_diff_local(x, y)
    (x + y) / two + (y - x) / two = y
    sinh(y) = sinh((x + y) / two) * cosh((y - x) / two) + cosh((x + y) / two) * sinh((y - x) / two)
    sinh_sub((x + y) / two, (y - x) / two)
    sinh((x + y) / two - (y - x) / two) =
        sinh((x + y) / two) * cosh((y - x) / two) - cosh((x + y) / two) * sinh((y - x) / two)
    half_sum_sub_half_diff_local(x, y)
    (x + y) / two - (y - x) / two = x
    sinh(x) = sinh((x + y) / two) * cosh((y - x) / two) - cosh((x + y) / two) * sinh((y - x) / two)
    sinh(y) - sinh(x) =
        (sinh((x + y) / two) * cosh((y - x) / two) + cosh((x + y) / two) * sinh((y - x) / two)) -
        (sinh((x + y) / two) * cosh((y - x) / two) - cosh((x + y) / two) * sinh((y - x) / two))
    sub_sum_diff(sinh((x + y) / two) * cosh((y - x) / two),
        cosh((x + y) / two) * sinh((y - x) / two))
    (sinh((x + y) / two) * cosh((y - x) / two) +
        cosh((x + y) / two) * sinh((y - x) / two)) -
        (sinh((x + y) / two) * cosh((y - x) / two) -
            cosh((x + y) / two) * sinh((y - x) / two)) =
        two * (cosh((x + y) / two) * sinh((y - x) / two))
    two * (cosh((x + y) / two) * sinh((y - x) / two)) =
        two * cosh((x + y) / two) * sinh((y - x) / two)
    sinh(y) - sinh(x) = two * cosh((x + y) / two) * sinh((y - x) / two)
}

/// The hyperbolic sine is strictly increasing.
theorem sinh_strictly_increasing(x: Real, y: Real) {
    x < y implies sinh(x) < sinh(y)
} by {
    if x < y {
        lt_imp_minus_pos(x, y)
        (y - x).is_positive
        pos_gt_zero(y - x)
        y - x > Real.0
        two_positive
        two > Real.0
        div_lt_div_pos(Real.0, y - x, two)
        Real.0 / two < (y - x) / two
        Real.0 / two = Real.0
        Real.0 < (y - x) / two
        sinh_pos_of_pos((y - x) / two)
        sinh((y - x) / two) > Real.0
        cosh_pos((x + y) / two)
        cosh((x + y) / two) > Real.0
        sinh_sub_formula(x, y)
        sinh(y) - sinh(x) = two * cosh((x + y) / two) * sinh((y - x) / two)
        two_positive
        two.is_positive
        cosh_pos((x + y) / two)
        cosh((x + y) / two) > Real.0
        pos_gt_zero(cosh((x + y) / two))
        cosh((x + y) / two).is_positive
        mul_pos_pos(two, cosh((x + y) / two))
        (two * cosh((x + y) / two)).is_positive
        pos_gt_zero(two * cosh((x + y) / two))
        two * cosh((x + y) / two) > Real.0
        sinh_pos_of_pos((y - x) / two)
        sinh((y - x) / two) > Real.0
        pos_gt_zero(sinh((y - x) / two))
        sinh((y - x) / two).is_positive
        mul_pos_pos(two * cosh((x + y) / two), sinh((y - x) / two))
        (two * cosh((x + y) / two) * sinh((y - x) / two)).is_positive
        pos_gt_zero(two * cosh((x + y) / two) * sinh((y - x) / two))
        two * cosh((x + y) / two) * sinh((y - x) / two) > Real.0
        sinh(y) - sinh(x) > Real.0
        lt_add_right(Real.0, sinh(y) - sinh(x), sinh(x))
        Real.0 + sinh(x) < (sinh(y) - sinh(x)) + sinh(x)
        Real.0 + sinh(x) = sinh(x)
        (sinh(y) - sinh(x)) + sinh(x) = sinh(y)
        sinh(x) < sinh(y)
    }
}

/// Equal hyperbolic sines have equal arguments.
theorem sinh_injective(x: Real, y: Real) {
    sinh(x) = sinh(y) implies x = y
} by {
    if sinh(x) = sinh(y) {
        if x < y {
            sinh_strictly_increasing(x, y)
            sinh(x) < sinh(y)
            false
        }
        if y < x {
            sinh_strictly_increasing(y, x)
            sinh(y) < sinh(x)
            false
        }
        not x < y
        not_lt_imp_gte(x, y)
        y <= x
        not y < x
        not_lt_imp_gte(y, x)
        x <= y
        lte_antisymm(x, y)
        x = y
    }
}

// Surjectivity of sinh.
//
// sinh is continuous (from differentiability), strictly increasing, and
// unbounded above and below because Real.exp is unbounded in both directions.
// Hence every real number is a sinh value by the intermediate value theorem.

/// The hyperbolic sine is continuous.
theorem sinh_continuous {
    continuous(sinh)
} by {
    sinh_is_derivative_fn
    is_derivative_fn_imp_continuous(sinh, cosh)
    continuous(sinh)
}

/// The hyperbolic sine of zero is zero.
theorem sinh_zero {
    sinh(Real.0) = Real.0
} by {
    sinh(Real.0) = ((Real.0).exp - (-Real.0).exp) / two
    neg_zero
    -Real.0 = Real.0
    (-Real.0).exp = (Real.0).exp
    exp_zero
    (Real.0).exp = Real.1
    (Real.0).exp - (-Real.0).exp = Real.1 - Real.1
    Real.1 - Real.1 = Real.0
    sinh(Real.0) = Real.0 / two
    Real.0 / two = Real.0
    sinh(Real.0) = Real.0
}

/// The hyperbolic sine is unbounded above.
theorem sinh_unbounded_above(y: Real) {
    exists(x: Real) {
        y < sinh(x)
    }
} by {
    if y < Real.0 {
        sinh_zero
        sinh(Real.0) = Real.0
        y < sinh(Real.0)
        exists(witness: Real) {
            witness = Real.0 and y < sinh(witness)
        }
        exists(b2: Real) { y < sinh(b2) }
    } else {
        not y < Real.0
        not_lt_imp_gte(y, Real.0)
        Real.0 <= y
        two_positive
        y >= Real.0
        two >= Real.0
        lte_self(Real.0)
        Real.0 >= Real.0
        lte_self(two)
        two <= two
        mul_le_mul_nonneg(Real.0, two, y, two)
        Real.0 * two <= y * two
        mul_zero_left(two)
        Real.0 * two = Real.0
        Real.0 <= y * two
        lt_add_one(y * two)
        y * two < y * two + Real.1
        exp_unbounded(y * two + Real.1)
        let b: Real satisfy {
            y * two + Real.1 < b.exp
        }
        y * two + Real.1 < b.exp
        exp_zero
        (Real.0).exp = Real.1
        lt_add_one(y * two)
        y * two < y * two + Real.1
        lt_of_lte_of_lt(Real.0, y * two, y * two + Real.1)
        Real.0 < y * two + Real.1
        lt_trans(Real.0, y * two + Real.1, b.exp)
        Real.0 < b.exp
        if not Real.0 < b {
            not_lt_imp_gte(Real.0, b)
            Real.0 >= b
            b <= Real.0
            if b = Real.0 {
                b.exp = (Real.0).exp
                b.exp = Real.1
                y * two + Real.1 < b.exp
                y * two + Real.1 < Real.1
                lt_add_right(y * two, Real.1, -Real.1)
                y * two + Real.1 + -Real.1 < Real.1 + -Real.1
                add_assoc(y * two, Real.1, -Real.1)
                (y * two + Real.1) + -Real.1 = y * two + (Real.1 + -Real.1)
                Real.1 + -Real.1 = Real.0
                y * two + Real.0 = y * two
                y * two + Real.1 + -Real.1 = y * two
                Real.1 + -Real.1 = Real.0
                y * two < Real.0
                Real.0 <= y * two
                false
            }
            b != Real.0
            not b = Real.0
            if not b < Real.0 {
                not_lt_imp_gte(b, Real.0)
                b >= Real.0
                Real.0 <= b
                b <= Real.0
                lte_antisymm(Real.0, b)
                Real.0 = b
                b = Real.0
                not b = Real.0
                false
            }
            b < Real.0
            exp_increasing(b, Real.0)
            b.exp < (Real.0).exp
            b.exp < Real.1
            lt_trans(y * two + Real.1, b.exp, Real.1)
            y * two + Real.1 < Real.1
            lt_add_right(y * two, Real.1, -Real.1)
            y * two + Real.1 + -Real.1 < Real.1 + -Real.1
            add_assoc(y * two, Real.1, -Real.1)
            (y * two + Real.1) + -Real.1 = y * two + (Real.1 + -Real.1)
            Real.1 + -Real.1 = Real.0
            y * two + Real.0 = y * two
            y * two + Real.1 + -Real.1 = y * two
            Real.1 + -Real.1 = Real.0
            y * two < Real.0
            Real.0 <= y * two
            false
        }
        Real.0 < b
        gt_zero_imp_pos(b)
        b.is_positive
        neg_pos_is_neg(b)
        -b < Real.0
        exp_increasing(-b, Real.0)
        (-b).exp < (Real.0).exp
        (-b).exp < Real.1
        lt_add_right((-b).exp, Real.1, -(-b).exp)
        (-b).exp + -(-b).exp < Real.1 + -(-b).exp
        (-b).exp + -(-b).exp = Real.0
        Real.0 < Real.1 + -(-b).exp
        Real.1 + -(-b).exp = Real.1 - (-b).exp
        Real.0 < Real.1 - (-b).exp
        lt_add_right(Real.0, Real.1 - (-b).exp, b.exp - Real.1)
        Real.0 + (b.exp - Real.1) < (Real.1 - (-b).exp) + (b.exp - Real.1)
        Real.0 + (b.exp - Real.1) = b.exp - Real.1
        add_sub_swap(Real.1, (-b).exp, b.exp)
        (Real.1 - (-b).exp) + (b.exp - Real.1) = b.exp - (-b).exp
        b.exp - Real.1 < b.exp - (-b).exp
        b.exp - (-b).exp > b.exp - Real.1
        y * two + Real.1 < b.exp
        lt_add_right(y * two + Real.1, b.exp, -Real.1)
        (y * two + Real.1) + -Real.1 < b.exp + -Real.1
        add_assoc(y * two, Real.1, -Real.1)
        (y * two + Real.1) + -Real.1 = y * two + (Real.1 + -Real.1)
        Real.1 + -Real.1 = Real.0
        y * two + Real.0 = y * two
        (y * two + Real.1) + -Real.1 = y * two
        b.exp + -Real.1 = b.exp - Real.1
        y * two < b.exp - Real.1
        b.exp - Real.1 > y * two
        lt_trans(y * two, b.exp - Real.1, b.exp - (-b).exp)
        y * two < b.exp - (-b).exp
        b.exp - (-b).exp > y * two
        div_lt_div_pos(y * two, b.exp - (-b).exp, two)
        y * two / two < (b.exp - (-b).exp) / two
        two_nonzero
        two != Real.0
        mul_div_cancel(y, two)
        (y * two) / two = y
        y < (b.exp - (-b).exp) / two
        sinh(b) = (b.exp - (-b).exp) / two
        y < sinh(b)
        exists(witness: Real) {
            witness = b and y < sinh(witness)
        }
        exists(b2: Real) { y < sinh(b2) }
    }
}


/// The hyperbolic sine is unbounded below.
theorem sinh_unbounded_below(y: Real) {
    exists(x: Real) {
        sinh(x) < y
    }
} by {
    sinh_unbounded_above(-y)
    let b: Real satisfy {
        -y < sinh(b)
    }
    -y < sinh(b)
    lt_imp_minus_pos(-y, sinh(b))
    (sinh(b) - -y).is_positive
    pos_gt_zero(sinh(b) - -y)
    sinh(b) - -y > Real.0
    neg_neg(y)
    -(-y) = y
    sinh(b) + y > Real.0
    lt_add_right(Real.0, sinh(b) + y, -sinh(b))
    Real.0 + -sinh(b) < (sinh(b) + y) + -sinh(b)
    Real.0 + -sinh(b) = -sinh(b)
    (sinh(b) + y) + -sinh(b) = (y + sinh(b)) + -sinh(b)
    (y + sinh(b)) + -sinh(b) = y + (sinh(b) + -sinh(b))
    sinh(b) + -sinh(b) = Real.0
    y + (sinh(b) + -sinh(b)) = y + Real.0
    y + Real.0 = y
    (sinh(b) + y) + -sinh(b) = y
    -sinh(b) < y
    sinh_neg(b)
    sinh(-b) = -sinh(b)
    sinh(-b) < y
    exists(witness: Real) {
        witness = -b and sinh(witness) < y
    }
}

/// Every real number is a hyperbolic sine value.
theorem exists_sinh_of(x: Real) {
    exists(y: Real) {
        sinh(y) = x
    }
} by {
    sinh_unbounded_below(x)
    let a: Real satisfy {
        sinh(a) < x
    }
    sinh_unbounded_above(x)
    let b: Real satisfy {
        x < sinh(b)
    }
    sinh(a) < x
    x < sinh(b)
    lt_trans(sinh(a), x, sinh(b))
    sinh(a) < sinh(b)
    if not a < b {
        not_lt_imp_gte(a, b)
        b <= a
        if b = a {
            sinh(b) = sinh(a)
            sinh(a) < sinh(b)
            false
        }
        b != a
        not b = a
        b < a
        sinh_strictly_increasing(b, a)
        sinh(b) < sinh(a)
        sinh(a) < sinh(b)
        false
    }
    a < b
    lt_imp_lte(a, b)
    a <= b
    sinh_continuous
    continuous(sinh)
    lt_imp_lte(sinh(a), x)
    sinh(a) <= x
    lt_imp_lte(x, sinh(b))
    x <= sinh(b)
    intermediate_value_closed_interval(sinh, a, b, x)
    let c: Real satisfy {
        closed_interval_set(a, b).contains(c) and sinh(c) = x
    }
    closed_interval_set(a, b).contains(c)
    sinh(c) = x
    exists(witness: Real) {
        witness = c and sinh(witness) = x
    }
    exists(y: Real) {
        sinh(y) = x
    }
}

// The inverse hyperbolic sine.
//
// Following the pattern of log in log.ac and arctan/arcsin in inverse_trig.ac,
// the arsinh is the default-valued witness of the hyperbolic-sine preimage
// predicate.

/// True if y is an inverse hyperbolic sine of x: sinh(y) = x.
define is_arsinh(x: Real, y: Real) -> Bool {
    sinh(y) = x
}

/// The inverse hyperbolic sine: the unique y with sinh(y) = x.
define arsinh(x: Real) -> Real {
    choose_or_default(is_arsinh(x), Real.0)
}

/// The hyperbolic-sine preimage of every real is unique.
theorem is_arsinh_unique(x: Real, y: Real, z: Real) {
    is_arsinh(x, y) and is_arsinh(x, z) implies y = z
} by {
    if is_arsinh(x, y) and is_arsinh(x, z) {
        is_arsinh(x, y) = (sinh(y) = x)
        sinh(y) = x
        is_arsinh(x, z) = (sinh(z) = x)
        sinh(z) = x
        sinh(y) = sinh(z)
        sinh_injective(y, z)
        y = z
    }
}

/// Every real has a unique inverse hyperbolic sine.
theorem exists_unique_arsinh(x: Real) {
    exists_unique(is_arsinh(x))
} by {
    exists_sinh_of(x)
    let y: Real satisfy {
        sinh(y) = x
    }
    sinh(y) = x
    is_arsinh(x, y)
    forall(z: Real) {
        if is_arsinh(x, z) {
            is_arsinh(x, z) = (sinh(z) = x)
            sinh(z) = x
            sinh(z) = sinh(y)
            sinh_injective(z, y)
            z = y
            y = z
        }
    }
    exists_unique_intro(is_arsinh(x), y)
    exists_unique(is_arsinh(x))
}

/// The inverse hyperbolic sine is a hyperbolic-sine preimage.
theorem is_arsinh_spec(x: Real) {
    is_arsinh(x, arsinh(x))
} by {
    exists_unique_arsinh(x)
    choose_or_default_spec(is_arsinh(x), Real.0)
    is_arsinh(x, choose_or_default(is_arsinh(x), Real.0))
    arsinh(x) = choose_or_default(is_arsinh(x), Real.0)
    is_arsinh(x, arsinh(x))
}

/// Sine of the inverse hyperbolic sine is the identity.
theorem sinh_arsinh(x: Real) {
    sinh(arsinh(x)) = x
} by {
    is_arsinh_spec(x)
    is_arsinh(x, arsinh(x))
    is_arsinh(x, arsinh(x)) = (sinh(arsinh(x)) = x)
    sinh(arsinh(x)) = x
}

/// The closed form of the inverse hyperbolic sine: arsinh(x) = ln(x + (x^2 + 1).sqrt).
theorem arsinh_log_form(x: Real, s: Real) {
    (x * x + Real.1).sqrt = Option.some(s) implies arsinh(x) = (x + s).log.get_or_else(Real.0)
} by {
    if (x * x + Real.1).sqrt = Option.some(s) {
        square_nonneg(x)
        x * x >= Real.0
        lt_add_pos(x * x, Real.1)
        x * x < x * x + Real.1
        lt_imp_lte(x * x, x * x + Real.1)
        x * x <= x * x + Real.1
        lte_trans(Real.0, x * x, x * x + Real.1)
        Real.0 <= x * x + Real.1
        x * x + Real.1 >= Real.0
        sqrt_value_mul_self(x * x + Real.1, s)
        s * s = x * x + Real.1
        sqrt_value_nonneg(x * x + Real.1, s)
        s >= Real.0
        // s > 0 since s*s = x*x + 1 >= 1 and s >= 0.
        lt_add_pos(x * x, Real.1)
        x * x < x * x + Real.1
        s * s = x * x + Real.1
        lt_trans(x * x, x * x + Real.1, s * s)
        x * x < s * s
        if s = Real.0 {
            s * s = Real.0 * Real.0
            Real.0 * Real.0 = Real.0
            s * s = Real.0
            lt_trans(x * x, x * x + Real.1, s * s)
            x * x < s * s
            false
        }
        s != Real.0
        not_lte_imp_gt(s, Real.0)
        s > Real.0
        // x + s > 0: if s <= -x then s*s <= x*x, contradicting s*s > x*x.
        if not x + s > Real.0 {
            not_lt_imp_gte(x + s, Real.0)
            x + s <= Real.0
            if x + s = Real.0 {
                s * s = x * x + Real.1
                x + s = Real.0
                x = -s
                x * x = s * s
                s * s = s * s + Real.1
                s * s = s * s
                Real.1 = Real.0
                zero_is_different_than_one
                false
            }
            x + s != Real.0
            x + s < Real.0
            lt_add_right(x + s, Real.0, -x)
            x + s + -x < Real.0 + -x
            add_swap_cancel(x, s)
            x + s + -x = s
            Real.0 + -x = -x
            s < -x
            lt_imp_lte(s, -x)
            s <= -x
            s >= Real.0
            -x > s
            lt_trans(Real.0, s, -x)
            Real.0 < -x
            lt_imp_lte(Real.0, -x)
            -x >= Real.0
            mul_le_mul_nonneg(s, s, -x, -x)
            s * s <= (-x) * (-x)
            mul_neg_left(x, -x)
            x * (-x) = -(x * x)
            mul_neg_right(x, x)
            x * (-x) = -(x * x)
            neg_neg(x * x)
            -(-(x * x)) = x * x
            (-x) * (-x) = x * x
            s * s <= x * x
            x * x < s * s
            false
        }
        x + s > Real.0
        log_some_of_pos_exists(x + s)
        exists(y: Real) { (x + s).log = Option.some(y) }
        let y: Real satisfy {
            (x + s).log = Option.some(y)
        }
        option_get_or_else_some[Real](y, Real.0)
        option_get_or_else(Option.some(y), Real.0) = y
        (x + s).log.get_or_else(Real.0) = y
        exp_log_or_zero(x + s, y)
        y.exp = x + s
        y.exp = x + s
        exp_neg(y)
        (-y).exp = Real.1 / y.exp
        y.exp = x + s
        Real.1 / y.exp = Real.1 / (x + s)
        (-(x + s).log.get_or_else(Real.0)).exp = Real.1 / (x + s)
        // 1/(x+s) = s - x from (x+s)(s-x) = 1.
        s * s - x * x = Real.1
        mul_add_sub_sq(x, s)
        (x + s) * (s - x) = s * s - x * x
        (x + s) * (s - x) = Real.1
        if x + s = Real.0 {
            (x + s) * (s - x) = Real.0 * (s - x)
            Real.0 * (s - x) = Real.0
            (x + s) * (s - x) = Real.0
            Real.0 = Real.1
            zero_is_different_than_one
            false
        }
        x + s != Real.0
        div_mul_cancel_left(x + s, s - x)
        ((x + s) * (s - x)) / (x + s) = s - x
        (x + s) * (s - x) = Real.1
        Real.1 / (x + s) = s - x
        (-(x + s).log.get_or_else(Real.0)).exp = s - x
        sinh((x + s).log.get_or_else(Real.0)) = (((x + s).log.get_or_else(Real.0)).exp - (-(x + s).log.get_or_else(Real.0)).exp) / two
        ((x + s).log.get_or_else(Real.0)).exp - (-(x + s).log.get_or_else(Real.0)).exp = (x + s) - (s - x)
        add_sub_neg_diff(x, s)
        (x + s) - (s - x) = two * x
        sinh((x + s).log.get_or_else(Real.0)) = (two * x) / two
        two_nonzero
        two != Real.0
        div_mul_cancel_left(two, x)
        (two * x) / two = x
        sinh((x + s).log.get_or_else(Real.0)) = x
        is_arsinh(x, (x + s).log.get_or_else(Real.0))
        is_arsinh_spec(x)
        is_arsinh(x, arsinh(x))
        is_arsinh_unique(x, arsinh(x), (x + s).log.get_or_else(Real.0))
        arsinh(x) = (x + s).log.get_or_else(Real.0)
    }
}
