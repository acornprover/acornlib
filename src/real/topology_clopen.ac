from data.basic.set import Set, compl_of_compl_is_self
from real.real_field import Real
from real.topology import empty_set_is_closed, empty_set_is_open, interior, is_closed_set,
    is_open_set, universal_set_is_closed, universal_set_is_open
from real.topology_closed_open import complement_of_open_is_closed, intersection_of_closed_is_closed,
    union_of_open_is_open
from real.topology_closure_union import union_of_closed_is_closed
from real.topology_complements import complement_of_closed_is_open
from real.topology_interior_algebra import open_set_eq_interior
from real.topology_interior_closure_idempotent import closed_set_iff_eq_closure,
    open_set_iff_eq_interior
from real.topology import closure
from real.topology_open_intersection import intersection_of_open_is_open

/// True if a real set is both open and closed.
define is_clopen_real_set(s: Set[Real]) -> Bool {
    is_open_set(s) and is_closed_set(s)
}

/// A clopen real set is open.
theorem clopen_real_set_is_open(s: Set[Real]) {
    is_clopen_real_set(s) implies is_open_set(s)
} by {
    if is_clopen_real_set(s) {
        is_clopen_real_set(s) = (is_open_set(s) and is_closed_set(s))
        is_open_set(s)
    }
}

/// A clopen real set is closed.
theorem clopen_real_set_is_closed(s: Set[Real]) {
    is_clopen_real_set(s) implies is_closed_set(s)
} by {
    if is_clopen_real_set(s) {
        is_clopen_real_set(s) = (is_open_set(s) and is_closed_set(s))
        is_closed_set(s)
    }
}

/// An open and closed real set is clopen.
theorem clopen_real_set_intro(s: Set[Real]) {
    is_open_set(s) and is_closed_set(s) implies is_clopen_real_set(s)
}

/// The empty real set is clopen.
theorem empty_real_set_is_clopen {
    is_clopen_real_set(Set[Real].empty_set)
} by {
    empty_set_is_open
    empty_set_is_closed
    is_open_set(Set[Real].empty_set)
    is_closed_set(Set[Real].empty_set)
    is_clopen_real_set(Set[Real].empty_set)
}

/// The universal real set is clopen.
theorem universal_real_set_is_clopen {
    is_clopen_real_set(Set[Real].universal_set)
} by {
    universal_set_is_open
    universal_set_is_closed
    is_open_set(Set[Real].universal_set)
    is_closed_set(Set[Real].universal_set)
    is_clopen_real_set(Set[Real].universal_set)
}

/// The complement of a clopen real set is clopen.
theorem complement_of_clopen_real_set_is_clopen(s: Set[Real]) {
    is_clopen_real_set(s) implies is_clopen_real_set(s.c)
} by {
    if is_clopen_real_set(s) {
        clopen_real_set_is_open(s)
        clopen_real_set_is_closed(s)
        complement_of_closed_is_open(s)
        complement_of_open_is_closed(s)
        is_open_set(s.c)
        is_closed_set(s.c)
        is_clopen_real_set(s.c)
    }
}

/// A real set is clopen if its complement is clopen.
theorem clopen_real_set_of_complement_clopen(s: Set[Real]) {
    is_clopen_real_set(s.c) implies is_clopen_real_set(s)
} by {
    if is_clopen_real_set(s.c) {
        complement_of_clopen_real_set_is_clopen(s.c)
        is_clopen_real_set(s.c.c)
        compl_of_compl_is_self[Real](s)
        s.c.c = s
        is_clopen_real_set(s)
    }
}

/// Clopenness is invariant under complement.
theorem complement_clopen_real_set_eq(s: Set[Real]) {
    is_clopen_real_set(s.c) = is_clopen_real_set(s)
} by {
    if is_clopen_real_set(s.c) {
        clopen_real_set_of_complement_clopen(s)
        is_clopen_real_set(s)
    }
    if is_clopen_real_set(s) {
        complement_of_clopen_real_set_is_clopen(s)
        is_clopen_real_set(s.c)
    }
    is_clopen_real_set(s.c) = is_clopen_real_set(s)
}

/// The intersection of two clopen real sets is clopen.
theorem intersection_of_clopen_real_sets_is_clopen(s: Set[Real], t: Set[Real]) {
    is_clopen_real_set(s) and is_clopen_real_set(t) implies is_clopen_real_set(s.intersection(t))
} by {
    if is_clopen_real_set(s) and is_clopen_real_set(t) {
        clopen_real_set_is_open(s)
        clopen_real_set_is_open(t)
        clopen_real_set_is_closed(s)
        clopen_real_set_is_closed(t)
        is_open_set(s)
        is_open_set(t)
        is_closed_set(s)
        is_closed_set(t)
        intersection_of_open_is_open(s, t)
        intersection_of_closed_is_closed(s, t)
        is_open_set(s.intersection(t))
        is_closed_set(s.intersection(t))
        is_clopen_real_set(s.intersection(t))
    }
}

/// The union of two clopen real sets is clopen.
theorem union_of_clopen_real_sets_is_clopen(s: Set[Real], t: Set[Real]) {
    is_clopen_real_set(s) and is_clopen_real_set(t) implies is_clopen_real_set(s.union(t))
} by {
    if is_clopen_real_set(s) and is_clopen_real_set(t) {
        clopen_real_set_is_open(s)
        clopen_real_set_is_open(t)
        clopen_real_set_is_closed(s)
        clopen_real_set_is_closed(t)
        is_open_set(s)
        is_open_set(t)
        is_closed_set(s)
        is_closed_set(t)
        union_of_open_is_open(s, t)
        union_of_closed_is_closed(s, t)
        is_open_set(s.union(t))
        is_closed_set(s.union(t))
        is_clopen_real_set(s.union(t))
    }
}

/// A clopen real set is equal to its interior.
theorem clopen_real_set_eq_interior(s: Set[Real]) {
    is_clopen_real_set(s) implies s = interior(s)
} by {
    if is_clopen_real_set(s) {
        clopen_real_set_is_open(s)
        open_set_eq_interior(s)
        s = interior(s)
    }
}

/// A clopen real set is equal to its closure.
theorem clopen_real_set_eq_closure(s: Set[Real]) {
    is_clopen_real_set(s) implies s = closure(s)
} by {
    if is_clopen_real_set(s) {
        clopen_real_set_is_closed(s)
        closed_set_iff_eq_closure(s)
        s = closure(s)
    }
}

/// A real set is clopen if it is fixed by both interior and closure.
theorem clopen_real_set_of_eq_interior_and_closure(s: Set[Real]) {
    s = interior(s) and s = closure(s) implies is_clopen_real_set(s)
} by {
    if s = interior(s) and s = closure(s) {
        open_set_iff_eq_interior(s)
        is_open_set(s)
        closed_set_iff_eq_closure(s)
        is_closed_set(s)
        is_clopen_real_set(s)
    }
}

/// Clopenness is equivalent to being fixed by interior and closure.
theorem clopen_real_set_eq_fixed_interior_closure(s: Set[Real]) {
    is_clopen_real_set(s) = (s = interior(s) and s = closure(s))
} by {
    if is_clopen_real_set(s) {
        clopen_real_set_eq_interior(s)
        clopen_real_set_eq_closure(s)
        s = interior(s) and s = closure(s)
    }
    if s = interior(s) and s = closure(s) {
        clopen_real_set_of_eq_interior_and_closure(s)
        is_clopen_real_set(s)
    }
    is_clopen_real_set(s) = (s = interior(s) and s = closure(s))
}

/// The interior of a clopen real set is clopen.
theorem interior_of_clopen_real_set_is_clopen(s: Set[Real]) {
    is_clopen_real_set(s) implies is_clopen_real_set(interior(s))
} by {
    if is_clopen_real_set(s) {
        clopen_real_set_eq_interior(s)
        s = interior(s)
        is_clopen_real_set(interior(s))
    }
}

/// The closure of a clopen real set is clopen.
theorem closure_of_clopen_real_set_is_clopen(s: Set[Real]) {
    is_clopen_real_set(s) implies is_clopen_real_set(closure(s))
} by {
    if is_clopen_real_set(s) {
        clopen_real_set_eq_closure(s)
        s = closure(s)
        is_clopen_real_set(closure(s))
    }
}

/// The interior of the intersection of two clopen real sets is that intersection.
theorem interior_intersection_of_clopen_real_sets(s: Set[Real], t: Set[Real]) {
    is_clopen_real_set(s) and is_clopen_real_set(t) implies interior(s.intersection(t)) = s.intersection(t)
} by {
    if is_clopen_real_set(s) and is_clopen_real_set(t) {
        intersection_of_clopen_real_sets_is_clopen(s, t)
        is_clopen_real_set(s.intersection(t))
        clopen_real_set_eq_interior(s.intersection(t))
        s.intersection(t) = interior(s.intersection(t))
        interior(s.intersection(t)) = s.intersection(t)
    }
}

/// The interior of the union of two clopen real sets is that union.
theorem interior_union_of_clopen_real_sets(s: Set[Real], t: Set[Real]) {
    is_clopen_real_set(s) and is_clopen_real_set(t) implies interior(s.union(t)) = s.union(t)
} by {
    if is_clopen_real_set(s) and is_clopen_real_set(t) {
        union_of_clopen_real_sets_is_clopen(s, t)
        is_clopen_real_set(s.union(t))
        clopen_real_set_eq_interior(s.union(t))
        s.union(t) = interior(s.union(t))
        interior(s.union(t)) = s.union(t)
    }
}
