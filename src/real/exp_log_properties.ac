from order import lt_of_lt_of_lte
from real.am_gm import exp_gt_one_plus_x_of_ne_zero, log_lt_sub_one
from real.exp import Real, exp_pos
from real.exp_inequalities import exp_ge_one_plus_self
from real.log import exp_log_or_zero, log_exp, log_some_of_pos_exists
from real.log_inequalities import log_strict_monotone
from real.real_base import lt_add_right
from real.real_ring import real_mul_comm

numerals Real

/// The exponential lies strictly above its tangent line `1 + x` away from zero.
theorem exp_gt_one_plus_x_strict(x: Real) {
    x != Real.0 implies x.exp > Real.1 + x
} by {
    if x != Real.0 {
        exp_gt_one_plus_x_of_ne_zero(x)
        x.exp > Real.1 + x
    }
}

/// The logarithm is strictly increasing on positive reals.
theorem log_strictly_increasing(x: Real, y: Real) {
    x > Real.0 and y > Real.0 and x < y implies x.log.get_or_else(Real.0) < y.log.get_or_else(Real.0)
} by {
    if x > Real.0 and y > Real.0 and x < y {
        log_some_of_pos_exists(x)
        exists(z: Real) { x.log = Option.some(z) }
        let z: Real satisfy {
            x.log = Option.some(z)
        }
        log_some_of_pos_exists(y)
        exists(w: Real) { y.log = Option.some(w) }
        let w: Real satisfy {
            y.log = Option.some(w)
        }
        log_strict_monotone(x, y, z, w)
        z < w
        option_get_or_else_some[Real](z, Real.0)
        option_get_or_else(Option.some(z), Real.0) = z
        x.log.get_or_else(Real.0) = z
        option_get_or_else_some[Real](w, Real.0)
        option_get_or_else(Option.some(w), Real.0) = w
        y.log.get_or_else(Real.0) = w
        z < w
        x.log.get_or_else(Real.0) < y.log.get_or_else(Real.0)
    }
}

/// An equality of exponential values is an equality of logarithms.
theorem exp_eq_log(x: Real, y: Real) {
    x.exp = y implies y.log = Option.some(x)
} by {
    if x.exp = y {
        log_exp(x)
        (x.exp).log = Option.some(x)
        y.log = Option.some(x)
    }
}

/// A logarithm equal to `x` exponentiates back to the base for positive inputs.
theorem log_eq_exp(x: Real, y: Real) {
    y > Real.0 and y.log = Option.some(x) implies x.exp = y
} by {
    if y > Real.0 and y.log = Option.some(x) {
        exp_log_or_zero(y, x)
        x.exp = y
    }
}

/// The exponential and the logarithm are inverse functions on positive reals.
theorem exp_eq_iff_log(x: Real, y: Real) {
    y > Real.0 implies (x.exp = y iff y.log = Option.some(x))
} by {
    if y > Real.0 {
        if x.exp = y {
            exp_eq_log(x, y)
            y.log = Option.some(x)
        }
        if y.log = Option.some(x) {
            log_eq_exp(x, y)
            x.exp = y
        }
        x.exp = y iff y.log = Option.some(x)
    }
}

/// Real powers of an exponential: `(e^x)^y = e^(x * y)`.
theorem exp_rpow_mul(x: Real, y: Real) {
    (x.exp).rpow(y) = Option.some((x * y).exp)
} by {
    exp_pos(x)
    log_exp(x)
    (x.exp).rpow(y) = Option.some((y * (x.exp).log.get_or_else(Real.0)).exp)
    (x.exp).log = Option.some(x)
    (x.exp).log.get_or_else(Real.0) = x
    (y * (x.exp).log.get_or_else(Real.0)).exp = (y * x).exp
    real_mul_comm(y, x)
    y * x = x * y
    (y * x).exp = (x * y).exp
    Option.some((y * (x.exp).log.get_or_else(Real.0)).exp) = Option.some((x * y).exp)
    (x.exp).rpow(y) = Option.some((x * y).exp)
}

/// The logarithm of `1 + x` is strictly below `x` for positive `x`.
theorem log_one_plus_x_lt_x(x: Real) {
    x > Real.0 implies (Real.1 + x).log.get_or_else(Real.0) < x
} by {
    if x > Real.0 {
        Real.1 + x > Real.0
        log_some_of_pos_exists(Real.1 + x)
        exists(z: Real) { (Real.1 + x).log = Option.some(z) }
        let z: Real satisfy {
            (Real.1 + x).log = Option.some(z)
        }
        Real.1 + x != Real.1
        log_lt_sub_one(Real.1 + x, z)
        z < Real.1 + x - Real.1
        Real.1 + x - Real.1 = x
        z < x
        option_get_or_else_some[Real](z, Real.0)
        option_get_or_else(Option.some(z), Real.0) = z
        (Real.1 + x).log.get_or_else(Real.0) = z
        z < x
        (Real.1 + x).log.get_or_else(Real.0) < x
    }
}

/// Every real is strictly below its exponential: `x < e^x`.
theorem x_lt_exp(x: Real) {
    x < x.exp
} by {
    exp_ge_one_plus_self(x)
    x.exp >= Real.1 + x
    Real.1 > Real.0
    Real.0 < Real.1
    lt_add_right(Real.0, Real.1, x)
    Real.0 + x < Real.1 + x
    Real.0 + x = x
    x < Real.1 + x
    lt_of_lt_of_lte(x, Real.1 + x, x.exp)
    x < x.exp
}
