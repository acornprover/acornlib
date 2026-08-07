from nat import Nat
from data.basic.set import Set
from real.real_field import Real
from real.real_seq import converges_to, tail_bound, tail_bound_implies_is_close
from real.sequence_limit_points import converges_to_has_tail_bound
from real.topology import is_closed_set, is_adherent_point_of_set, is_eps_adherent_to_set,
    eps_adherent_of_contains_close, adherent_point_intro, closed_set_contains_adherent

numerals Real

/// A contained convergent sequence supplies an epsilon-adherent witness.
theorem seq_converges_imp_eps_adherent(s: Set[Real], a: Nat -> Real, x: Real, eps: Real) {
    forall(n: Nat) { s.contains(a(n)) } and converges_to(a, x) and eps.is_positive
    implies is_eps_adherent_to_set(s, x, eps)
} by {
    if forall(n: Nat) { s.contains(a(n)) } and converges_to(a, x) and eps.is_positive {
        converges_to_has_tail_bound(a, x, eps)
        let big_n: Nat satisfy {
            tail_bound(a, x, big_n, eps)
        }
        tail_bound(a, x, big_n, eps)
        s.contains(a(big_n))
        tail_bound_implies_is_close(a, x, big_n, eps, big_n)
        a(big_n).is_close(x, eps)
        eps_adherent_of_contains_close(s, x, a(big_n), eps)
        is_eps_adherent_to_set(s, x, eps)
    }
}

/// A sequence in a set converging to `x` makes `x` adherent to that set.
theorem seq_converges_imp_adherent(s: Set[Real], a: Nat -> Real, x: Real) {
    forall(n: Nat) { s.contains(a(n)) } and converges_to(a, x)
    implies is_adherent_point_of_set(s, x)
} by {
    if forall(n: Nat) { s.contains(a(n)) } and converges_to(a, x) {
        forall(eps: Real) {
            if eps.is_positive {
                seq_converges_imp_eps_adherent(s, a, x, eps)
            }
        }
        adherent_point_intro(s, x)
        is_adherent_point_of_set(s, x)
    }
}

/// A closed set contains the limit of every convergent sequence it contains.
theorem closed_set_contains_seq_limit(s: Set[Real], a: Nat -> Real, x: Real) {
    is_closed_set(s) and forall(n: Nat) { s.contains(a(n)) } and converges_to(a, x)
    implies s.contains(x)
} by {
    if is_closed_set(s) and forall(n: Nat) { s.contains(a(n)) } and converges_to(a, x) {
        seq_converges_imp_adherent(s, a, x)
        is_adherent_point_of_set(s, x)
        closed_set_contains_adherent(s, x)
        s.contains(x)
    }
}
