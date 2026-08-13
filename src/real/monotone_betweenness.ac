/// Betweenness for continuous injective functions on intervals.
///
/// A continuous injective function sends the middle point of a strictly
/// ordered triple to a value between the two outer values.

from order import lte_trans, lt_trans, lt_imp_lte, lt_imp_ne, lt_imp_ne_symm,
    not_lt_imp_gte, lte_antisymm, lt_of_lte_of_lt, lt_of_lt_of_lte, not_lt_self
from order_set import closed_interval_set, closed_interval_set_lower_le,
    closed_interval_set_le_upper
from real.continuity_base import Real, continuous
from real.intermediate_value import intermediate_value_closed_interval
from real.monotone_base import injective_on, injective_on_apply,
    increasing_on_apply, intermediate_value_closed_interval_rev,
    closed_interval_inner_member

numerals Real

/// A continuous injective function keeps the order of any triple with a rising
/// left step: for x < y < z, if f(x) < f(y) then f(y) < f(z).
theorem continuous_injective_betweenness_rising(
    f: Real -> Real, a: Real, b: Real, x: Real, y: Real, z: Real
) {
    continuous(f) and injective_on(f, a, b) and
    closed_interval_set(a, b).contains(x) and closed_interval_set(a, b).contains(y) and
    closed_interval_set(a, b).contains(z) and x < y and y < z and f(x) < f(y)
    implies f(y) < f(z)
} by {
    if continuous(f) and injective_on(f, a, b) and
       closed_interval_set(a, b).contains(x) and closed_interval_set(a, b).contains(y) and
       closed_interval_set(a, b).contains(z) and x < y and y < z and f(x) < f(y) {
        closed_interval_set_lower_le(a, b, x)
        a <= x
        closed_interval_set_le_upper(a, b, z)
        z <= b
        closed_interval_set_le_upper(a, b, y)
        y <= b
        lt_trans[Real](x, y, z)
        x < z
        lt_imp_lte(x, z)
        x <= z
        lte_trans[Real](a, x, z)
        a <= z
        lt_imp_lte(x, y)
        x <= y
        lte_trans[Real](a, x, y)
        a <= y
        if not f(y) < f(z) {
            not_lt_imp_gte[Real](f(y), f(z))
            f(z) <= f(y)
            if f(x) < f(z) {
                if f(z) < f(y) {
                    lt_imp_lte(f(x), f(z))
                    f(x) <= f(z)
                    lt_imp_lte(f(z), f(y))
                    f(z) <= f(y)
                    intermediate_value_closed_interval(f, x, y, f(z))
                    let w: Real satisfy {
                        closed_interval_set(x, y).contains(w) and f(w) = f(z)
                    }
                    closed_interval_set(x, y).contains(w)
                    closed_interval_set_le_upper(x, y, w)
                    w <= y
                    lt_of_lte_of_lt[Real](w, y, z)
                    w < z
                    lt_imp_ne_symm(w, z)
                    w != z
                    closed_interval_inner_member(a, b, x, y, w)
                    closed_interval_set(a, b).contains(w)
                    f(w) = f(z)
                    injective_on_apply(f, a, b, w, z)
                    w = z
                    false
                } else {
                    not_lt_imp_gte[Real](f(z), f(y))
                    f(y) <= f(z)
                    lte_antisymm[Real](f(y), f(z))
                    f(y) = f(z)
                    injective_on_apply(f, a, b, y, z)
                    y = z
                    y < z
                    false
                }
            } else {
                not_lt_imp_gte[Real](f(x), f(z))
                f(z) <= f(x)
                if f(z) < f(x) {
                    lt_imp_lte(f(z), f(x))
                    f(z) <= f(x)
                    lt_imp_lte(f(x), f(y))
                    f(x) <= f(y)
                    lt_imp_lte(y, z)
                    y <= z
                    intermediate_value_closed_interval_rev(f, y, z, f(x))
                    let w: Real satisfy {
                        closed_interval_set(y, z).contains(w) and f(w) = f(x)
                    }
                    closed_interval_set(y, z).contains(w)
                    closed_interval_set_lower_le(y, z, w)
                    y <= w
                    lt_of_lt_of_lte[Real](x, y, w)
                    x < w
                    lt_imp_ne_symm(x, w)
                    w != x
                    closed_interval_inner_member(a, b, y, z, w)
                    closed_interval_set(a, b).contains(w)
                    f(w) = f(x)
                    injective_on_apply(f, a, b, w, x)
                    w = x
                    false
                } else {
                    not_lt_imp_gte[Real](f(z), f(x))
                    f(x) <= f(z)
                    lte_antisymm[Real](f(x), f(z))
                    f(x) = f(z)
                    injective_on_apply(f, a, b, x, z)
                    x = z
                    x < z
                    false
                }
            }
        }
        f(y) < f(z)
    }
}

/// A continuous injective function keeps the order of any triple with a falling
/// right step: for x < y < z, if f(z) < f(y) then f(y) < f(x).
theorem continuous_injective_betweenness_falling(
    f: Real -> Real, a: Real, b: Real, x: Real, y: Real, z: Real
) {
    continuous(f) and injective_on(f, a, b) and
    closed_interval_set(a, b).contains(x) and closed_interval_set(a, b).contains(y) and
    closed_interval_set(a, b).contains(z) and x < y and y < z and f(z) < f(y)
    implies f(y) < f(x)
} by {
    if continuous(f) and injective_on(f, a, b) and
       closed_interval_set(a, b).contains(x) and closed_interval_set(a, b).contains(y) and
       closed_interval_set(a, b).contains(z) and x < y and y < z and f(z) < f(y) {
        closed_interval_set_lower_le(a, b, x)
        a <= x
        closed_interval_set_le_upper(a, b, z)
        z <= b
        closed_interval_set_le_upper(a, b, y)
        y <= b
        lt_trans[Real](x, y, z)
        x < z
        lt_imp_lte(x, z)
        x <= z
        lte_trans[Real](a, x, z)
        a <= z
        lt_imp_lte(x, y)
        x <= y
        lte_trans[Real](a, x, y)
        a <= y
        if not f(y) < f(x) {
            not_lt_imp_gte[Real](f(y), f(x))
            f(x) <= f(y)
            if f(x) < f(z) {
                lt_imp_lte(f(x), f(z))
                f(x) <= f(z)
                lt_imp_lte(f(z), f(y))
                f(z) <= f(y)
                intermediate_value_closed_interval(f, x, y, f(z))
                let w: Real satisfy {
                    closed_interval_set(x, y).contains(w) and f(w) = f(z)
                }
                closed_interval_set(x, y).contains(w)
                closed_interval_set_le_upper(x, y, w)
                w <= y
                lt_of_lte_of_lt[Real](w, y, z)
                w < z
                lt_imp_ne_symm(w, z)
                w != z
                closed_interval_inner_member(a, b, x, y, w)
                closed_interval_set(a, b).contains(w)
                f(w) = f(z)
                injective_on_apply(f, a, b, w, z)
                w = z
                false
            } else {
                not_lt_imp_gte[Real](f(x), f(z))
                f(z) <= f(x)
                if f(z) < f(x) {
                    if f(x) < f(y) {
                        lt_imp_lte(f(z), f(x))
                        f(z) <= f(x)
                        lt_imp_lte(f(x), f(y))
                        f(x) <= f(y)
                        lt_imp_lte(y, z)
                        y <= z
                        intermediate_value_closed_interval_rev(f, y, z, f(x))
                        let w: Real satisfy {
                            closed_interval_set(y, z).contains(w) and f(w) = f(x)
                        }
                        closed_interval_set(y, z).contains(w)
                        closed_interval_set_lower_le(y, z, w)
                        y <= w
                        lt_of_lt_of_lte[Real](x, y, w)
                        x < w
                        lt_imp_ne_symm(x, w)
                        w != x
                        closed_interval_inner_member(a, b, y, z, w)
                        closed_interval_set(a, b).contains(w)
                        f(w) = f(x)
                        injective_on_apply(f, a, b, w, x)
                        w = x
                        false
                    } else {
                        not_lt_imp_gte[Real](f(x), f(y))
                        f(y) <= f(x)
                        lte_antisymm[Real](f(x), f(y))
                        f(x) = f(y)
                        injective_on_apply(f, a, b, x, y)
                        x = y
                        x < y
                        false
                    }
                } else {
                    not_lt_imp_gte[Real](f(z), f(x))
                    f(x) <= f(z)
                    lte_antisymm[Real](f(x), f(z))
                    f(x) = f(z)
                    injective_on_apply(f, a, b, x, z)
                    x = z
                    x < z
                    false
                }
            }
        }
        f(y) < f(x)
    }
}

