/// Bounded-factor consumers for geometrically vanishing real sequences.
///
/// This module packages the shallow downstream pattern: once a sequence is known
/// to vanish from geometric-majorant hypotheses, multiplying it by a bounded
/// real sequence still vanishes. It intentionally keeps the explicit-index
/// hypotheses from `real.series_geometric_term_vanishes` and does not introduce
/// asymptotic notation or a generic eventual API.

from nat import Nat
from real.real_base import Real
from real.abs_conv import abs_fn
from real.bounded_seq import is_bounded_seq
from real.limits import vanishes
from real.prod_seq import prod_seq
from real.real_series import tail
from real.series_geometric_majorant import geometric_majorant
from real.series_geometric_term_vanishes import geometric_majorant_terms_vanish,
    eventually_abs_le_geometric_terms_vanish, eventually_nonneg_le_geometric_terms_vanish,
    tail_abs_le_geometric_terms_vanish, tail_nonneg_le_geometric_terms_vanish
from real.asymptotic_bounds import bounded_mul_vanishing_seq, vanishing_mul_bounded_seq

numerals Real

/// A bounded factor times a geometric majorant has vanishing terms.
theorem bounded_mul_geometric_majorant_terms_vanish(a: Nat -> Real, c: Real, r: Real) {
    is_bounded_seq(a)
    and Real.0 <= r
    and r < Real.1
    implies vanishes(prod_seq(a, geometric_majorant(c, r)))
} by {
    if is_bounded_seq(a) and Real.0 <= r and r < Real.1 {
        geometric_majorant_terms_vanish(c, r)
        vanishes(geometric_majorant(c, r))
        bounded_mul_vanishing_seq(a, geometric_majorant(c, r))
        vanishes(prod_seq(a, geometric_majorant(c, r)))
    }
}

/// A geometric majorant times a bounded factor has vanishing terms.
theorem geometric_majorant_mul_bounded_terms_vanish(a: Nat -> Real, c: Real, r: Real) {
    is_bounded_seq(a)
    and Real.0 <= r
    and r < Real.1
    implies vanishes(prod_seq(geometric_majorant(c, r), a))
} by {
    if is_bounded_seq(a) and Real.0 <= r and r < Real.1 {
        geometric_majorant_terms_vanish(c, r)
        vanishes(geometric_majorant(c, r))
        vanishing_mul_bounded_seq(geometric_majorant(c, r), a)
        vanishes(prod_seq(geometric_majorant(c, r), a))
    }
}

/// A bounded factor times an eventually absolutely geometrically dominated sequence vanishes.
theorem bounded_mul_eventually_abs_le_geometric_terms_vanish(
    a: Nat -> Real,
    f: Nat -> Real,
    c: Real,
    r: Real,
    n: Nat
) {
    is_bounded_seq(a)
    and Real.0 <= c
    and Real.0 <= r
    and r < Real.1
    and (forall(k: Nat) { n <= k implies abs_fn(f)(k) <= geometric_majorant(c, r, k) })
    implies vanishes(prod_seq(a, f))
} by {
    if is_bounded_seq(a)
        and Real.0 <= c
        and Real.0 <= r
        and r < Real.1
        and (forall(k: Nat) { n <= k implies abs_fn(f)(k) <= geometric_majorant(c, r, k) }) {
        eventually_abs_le_geometric_terms_vanish(f, c, r, n)
        vanishes(f)
        bounded_mul_vanishing_seq(a, f)
        vanishes(prod_seq(a, f))
    }
}

/// An eventually absolutely geometrically dominated sequence times a bounded factor vanishes.
theorem eventually_abs_le_geometric_mul_bounded_terms_vanish(
    f: Nat -> Real,
    a: Nat -> Real,
    c: Real,
    r: Real,
    n: Nat
) {
    is_bounded_seq(a)
    and Real.0 <= c
    and Real.0 <= r
    and r < Real.1
    and (forall(k: Nat) { n <= k implies abs_fn(f)(k) <= geometric_majorant(c, r, k) })
    implies vanishes(prod_seq(f, a))
} by {
    if is_bounded_seq(a)
        and Real.0 <= c
        and Real.0 <= r
        and r < Real.1
        and (forall(k: Nat) { n <= k implies abs_fn(f)(k) <= geometric_majorant(c, r, k) }) {
        eventually_abs_le_geometric_terms_vanish(f, c, r, n)
        vanishes(f)
        vanishing_mul_bounded_seq(f, a)
        vanishes(prod_seq(f, a))
    }
}

/// A bounded factor times an eventually nonnegative geometrically dominated sequence vanishes.
theorem bounded_mul_eventually_nonneg_le_geometric_terms_vanish(
    a: Nat -> Real,
    f: Nat -> Real,
    c: Real,
    r: Real,
    n: Nat
) {
    is_bounded_seq(a)
    and Real.0 <= c
    and Real.0 <= r
    and r < Real.1
    and (forall(k: Nat) { n <= k implies Real.0 <= f(k) })
    and (forall(k: Nat) { n <= k implies f(k) <= geometric_majorant(c, r, k) })
    implies vanishes(prod_seq(a, f))
} by {
    if is_bounded_seq(a)
        and Real.0 <= c
        and Real.0 <= r
        and r < Real.1
        and (forall(k: Nat) { n <= k implies Real.0 <= f(k) })
        and (forall(k: Nat) { n <= k implies f(k) <= geometric_majorant(c, r, k) }) {
        eventually_nonneg_le_geometric_terms_vanish(f, c, r, n)
        vanishes(f)
        bounded_mul_vanishing_seq(a, f)
        vanishes(prod_seq(a, f))
    }
}

/// An eventually nonnegative geometrically dominated sequence times a bounded factor vanishes.
theorem eventually_nonneg_le_geometric_mul_bounded_terms_vanish(
    f: Nat -> Real,
    a: Nat -> Real,
    c: Real,
    r: Real,
    n: Nat
) {
    is_bounded_seq(a)
    and Real.0 <= c
    and Real.0 <= r
    and r < Real.1
    and (forall(k: Nat) { n <= k implies Real.0 <= f(k) })
    and (forall(k: Nat) { n <= k implies f(k) <= geometric_majorant(c, r, k) })
    implies vanishes(prod_seq(f, a))
} by {
    if is_bounded_seq(a)
        and Real.0 <= c
        and Real.0 <= r
        and r < Real.1
        and (forall(k: Nat) { n <= k implies Real.0 <= f(k) })
        and (forall(k: Nat) { n <= k implies f(k) <= geometric_majorant(c, r, k) }) {
        eventually_nonneg_le_geometric_terms_vanish(f, c, r, n)
        vanishes(f)
        vanishing_mul_bounded_seq(f, a)
        vanishes(prod_seq(f, a))
    }
}

/// A bounded factor times a sequence with an absolutely geometrically dominated tail vanishes.
theorem bounded_mul_tail_abs_le_geometric_terms_vanish(
    a: Nat -> Real,
    f: Nat -> Real,
    c: Real,
    r: Real,
    n: Nat
) {
    is_bounded_seq(a)
    and Real.0 <= c
    and Real.0 <= r
    and r < Real.1
    and (forall(k: Nat) { abs_fn(tail(f, n))(k) <= geometric_majorant(c, r, k) })
    implies vanishes(prod_seq(a, f))
} by {
    if is_bounded_seq(a)
        and Real.0 <= c
        and Real.0 <= r
        and r < Real.1
        and (forall(k: Nat) { abs_fn(tail(f, n))(k) <= geometric_majorant(c, r, k) }) {
        tail_abs_le_geometric_terms_vanish(f, c, r, n)
        vanishes(f)
        bounded_mul_vanishing_seq(a, f)
        vanishes(prod_seq(a, f))
    }
}

/// A sequence with an absolutely geometrically dominated tail times a bounded factor vanishes.
theorem tail_abs_le_geometric_mul_bounded_terms_vanish(
    f: Nat -> Real,
    a: Nat -> Real,
    c: Real,
    r: Real,
    n: Nat
) {
    is_bounded_seq(a)
    and Real.0 <= c
    and Real.0 <= r
    and r < Real.1
    and (forall(k: Nat) { abs_fn(tail(f, n))(k) <= geometric_majorant(c, r, k) })
    implies vanishes(prod_seq(f, a))
} by {
    if is_bounded_seq(a)
        and Real.0 <= c
        and Real.0 <= r
        and r < Real.1
        and (forall(k: Nat) { abs_fn(tail(f, n))(k) <= geometric_majorant(c, r, k) }) {
        tail_abs_le_geometric_terms_vanish(f, c, r, n)
        vanishes(f)
        vanishing_mul_bounded_seq(f, a)
        vanishes(prod_seq(f, a))
    }
}

/// A bounded factor times a sequence with a nonnegative geometrically dominated tail vanishes.
theorem bounded_mul_tail_nonneg_le_geometric_terms_vanish(
    a: Nat -> Real,
    f: Nat -> Real,
    c: Real,
    r: Real,
    n: Nat
) {
    is_bounded_seq(a)
    and Real.0 <= c
    and Real.0 <= r
    and r < Real.1
    and (forall(k: Nat) { Real.0 <= tail(f, n)(k) })
    and (forall(k: Nat) { tail(f, n)(k) <= geometric_majorant(c, r, k) })
    implies vanishes(prod_seq(a, f))
} by {
    if is_bounded_seq(a)
        and Real.0 <= c
        and Real.0 <= r
        and r < Real.1
        and (forall(k: Nat) { Real.0 <= tail(f, n)(k) })
        and (forall(k: Nat) { tail(f, n)(k) <= geometric_majorant(c, r, k) }) {
        tail_nonneg_le_geometric_terms_vanish(f, c, r, n)
        vanishes(f)
        bounded_mul_vanishing_seq(a, f)
        vanishes(prod_seq(a, f))
    }
}

/// A sequence with a nonnegative geometrically dominated tail times a bounded factor vanishes.
theorem tail_nonneg_le_geometric_mul_bounded_terms_vanish(
    f: Nat -> Real,
    a: Nat -> Real,
    c: Real,
    r: Real,
    n: Nat
) {
    is_bounded_seq(a)
    and Real.0 <= c
    and Real.0 <= r
    and r < Real.1
    and (forall(k: Nat) { Real.0 <= tail(f, n)(k) })
    and (forall(k: Nat) { tail(f, n)(k) <= geometric_majorant(c, r, k) })
    implies vanishes(prod_seq(f, a))
} by {
    if is_bounded_seq(a)
        and Real.0 <= c
        and Real.0 <= r
        and r < Real.1
        and (forall(k: Nat) { Real.0 <= tail(f, n)(k) })
        and (forall(k: Nat) { tail(f, n)(k) <= geometric_majorant(c, r, k) }) {
        tail_nonneg_le_geometric_terms_vanish(f, c, r, n)
        vanishes(f)
        vanishing_mul_bounded_seq(f, a)
        vanishes(prod_seq(f, a))
    }
}
