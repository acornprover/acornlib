/// Darboux's theorem: derivatives have the intermediate value property.
///
/// This file proves Darboux's theorem (Gaston Darboux, 1875): if a real
/// function `f` is differentiable everywhere and its pointwise derivative
/// `df` takes values `df(a) < t < df(b)` at the endpoints of an interval
/// `[a, b]`, then some interior point `c` has `df(c) = t`.  In other words,
/// every derivative has the intermediate value property, even though the
/// derivative itself need not be continuous.
///
/// The proof is the classical one.  Consider the auxiliary function
/// `g(x) = f(x) - t x`.  Its derivative at `x` is `df(x) - t`, so
/// `g'(a) = df(a) - t < 0` and `g'(b) = df(b) - t > 0`.  A negative
/// derivative at `a` forces `g` strictly below `g(a)` just to the right of
/// `a`, and a positive derivative at `b` forces `g` strictly below `g(b)`
/// just to the left of `b`; hence the minimum of `g` on `[a, b]` (which
/// exists by the extreme value theorem, since `g` is continuous) is not
/// attained at either endpoint, so it is attained at an interior point `c`.
/// By Fermat's theorem on interior extrema, `g'(c) = 0`, i.e. `df(c) = t`.

from order import lte_trans, lt_trans, lt_imp_lte, not_lt_imp_gte, lte_antisymm,
    lt_of_lt_of_lte, lt_of_lte_of_lt, lt_imp_ne, lt_imp_ne_symm, not_lt_self,
    not_lte_imp_gt, lte_imp_not_lt, lte_refl
from order_set import closed_interval_set, closed_interval_set_contains_eq,
    closed_interval_set_lower_le, closed_interval_set_le_upper
from order import closed_interval
from real.continuity_base import Real, continuous, continuous_at
from real.continuity_affine import affine_real, continuous_at_affine_real
from real.continuity_pointwise import continuous_at_pointwise_neg, continuous_at_pointwise_add
from real.derivative_basic import has_derivative_at, differentiable_at, difference_quotient,
    has_derivative_at_delta, forall_elim, sub_ne_zero_of_ne, has_derivative_at_unique
from real.derivative_rules import derivative_pointwise_neg, derivative_pointwise_sub,
    differentiable_pointwise_sub, neg_div
from real.derivative_affine_named import affine_real_has_derivative_at
from real.derivative_continuity import differentiable_continuous_at
from real.mean_value import continuous_on_closed, continuous_closed_interval_attains_minimum,
    fermat_interior_maximum, lte_imp_neg_lte_neg
from real.real_base import add_comm, close_imp_bounds, lt_add_pos, pos_imp_eq_abs,
    neg_zero, neg_distrib, add_assoc, self_close, sub_cancels, neg_neg, abs_neg, lt_add_right,
    lte_add_right, add_neg_eq_zero, gt_zero_imp_pos, add_zero_left, add_zero_right,
    neg_pos_is_neg, neg_lt_zero, pos_gt_zero, lt_add_converse, bounds_imp_close, lte_abs,
    abs_gte_zero, lte_lt_trans, lte_both_ways_imp_eq
from real.real_field import mul_inverse, mul_left_cancel, zero_is_different_than_one,
    div_lt_mul_pos, mul_div_cancel
from real.real_ring import mul_zero_left, mul_zero_right, real_mul_comm, mul_distrib_right,
    mul_neg_left, mul_neg_right, mul_one_right
from real.convex import neg_inverse
from real.real_seq import eps_smaller_than_both, close_and_lt_imp_close, eps_lt_half,
    neg_is_close, sub_zero_imp_eq, lt_imp_minus_pos
from data.basic.function_algebra import pointwise_neg, pointwise_add
from data.basic.functions import function_extensionality, function_eq_transport_predicate_rev

numerals Real

/// The auxiliary function `x -> f(x) - t * x` used in the proof of
/// Darboux's theorem.
define darboux_aux(f: Real -> Real, t: Real) -> (Real -> Real) {
    pointwise_add(f, pointwise_neg(affine_real(t, Real.0)))
}

/// The auxiliary function evaluates to `f(x) - t x`.
theorem darboux_aux_value(f: Real -> Real, t: Real, x: Real) {
    darboux_aux(f, t, x) = f(x) - t * x
} by {
    darboux_aux(f, t, x) = pointwise_add(f, pointwise_neg(affine_real(t, Real.0)), x)
    pointwise_add(f, pointwise_neg(affine_real(t, Real.0)), x) =
        f(x) + pointwise_neg(affine_real(t, Real.0), x)
    pointwise_neg(affine_real(t, Real.0), x) = -affine_real(t, Real.0, x)
    affine_real(t, Real.0, x) = t * x + Real.0
    t * x + Real.0 = t * x
    darboux_aux(f, t, x) = f(x) + -(t * x)
    f(x) - t * x = f(x) + -(t * x)
    darboux_aux(f, t, x) = f(x) - t * x
}

/// The derivative of the auxiliary function at `x` is `df(x) - t`.
theorem darboux_aux_has_derivative_at(f: Real -> Real, df: Real -> Real, t: Real, x: Real) {
    has_derivative_at(f, x, df(x))
    implies has_derivative_at(darboux_aux(f, t), x, df(x) - t)
} by {
    if has_derivative_at(f, x, df(x)) {
        affine_real_has_derivative_at(t, Real.0, x)
        has_derivative_at(affine_real(t, Real.0), x, t)
        derivative_pointwise_sub(f, affine_real(t, Real.0), x, df(x), t)
        has_derivative_at(pointwise_add(f, pointwise_neg(affine_real(t, Real.0))), x, df(x) + -t)
        has_derivative_at(darboux_aux(f, t), x, df(x) - t)
    }
}

/// The auxiliary function is continuous at every point where `f` is
/// differentiable.
theorem darboux_aux_continuous_at(f: Real -> Real, df: Real -> Real, t: Real, x: Real) {
    has_derivative_at(f, x, df(x))
    implies continuous_at(darboux_aux(f, t), x)
} by {
    if has_derivative_at(f, x, df(x)) {
        differentiable_at(f, x)
        differentiable_continuous_at(f, x)
        continuous_at(f, x)
        continuous_at_affine_real(t, Real.0, x)
        continuous_at(affine_real(t, Real.0), x)
        continuous_at_pointwise_neg(affine_real(t, Real.0), x)
        continuous_at(pointwise_neg(affine_real(t, Real.0)), x)
        continuous_at_pointwise_add(f, pointwise_neg(affine_real(t, Real.0)), x)
        continuous_at(pointwise_add(f, pointwise_neg(affine_real(t, Real.0))), x)
        continuous_at(darboux_aux(f, t), x)
    }
}

/// The auxiliary function is continuous on the closed interval `[a, b]`
/// when `f` is differentiable everywhere.
theorem darboux_aux_continuous_on_closed(
    f: Real -> Real, df: Real -> Real, t: Real, a: Real, b: Real
) {
    (forall(x: Real) { has_derivative_at(f, x, df(x)) })
    implies continuous_on_closed(darboux_aux(f, t), a, b)
} by {
    if forall(x: Real) { has_derivative_at(f, x, df(x)) } {
        forall(x: Real) {
            if closed_interval_set(a, b).contains(x) {
                has_derivative_at(f, x, df(x))
                darboux_aux_continuous_at(f, df, t, x)
                continuous_at(darboux_aux(f, t), x)
            }
        }
        continuous_on_closed(darboux_aux(f, t), a, b) = forall(x: Real) {
            closed_interval_set(a, b).contains(x) implies continuous_at(darboux_aux(f, t), x)
        }
        continuous_on_closed(darboux_aux(f, t), a, b)
    }
}

/// Division by a negated denominator negates the quotient.
theorem div_neg_denominator(a: Real, b: Real) {
    b != Real.0 implies a / (-b) = -(a / b)
} by {
    if b != Real.0 {
        neg_inverse(b)
        (-b).inverse = -b.inverse
        a / (-b) = a * (-b).inverse
        a * (-b).inverse = a * -b.inverse
        mul_neg_right(a, b.inverse)
        a * -b.inverse = -(a * b.inverse)
        a * b.inverse = a / b
        a / (-b) = -(a / b)
    }
}

/// A negative derivative at `a` forces the function strictly below its
/// value at `a` just to the right of `a`.
theorem derivative_negative_imp_dips_right(h: Real -> Real, a: Real, b: Real, d: Real) {
    a < b and has_derivative_at(h, a, d) and d < Real.0
    implies exists(x: Real) { a < x and x < b and h(x) < h(a) }
} by {
    if a < b and has_derivative_at(h, a, d) and d < Real.0 {
        // The tolerance -d is positive.
        lt_add_right(d, Real.0, -d)
        d + -d < Real.0 + -d
        add_neg_eq_zero(d)
        d + -d = Real.0
        add_zero_left(-d)
        Real.0 + -d = -d
        Real.0 < -d
        gt_zero_imp_pos(-d)
        (-d).is_positive
        has_derivative_at_delta(h, a, d, -d)
        let delta: Real satisfy {
            delta.is_positive and forall(x: Real) {
                x != a and x.is_close(a, delta)
                implies difference_quotient(h, a, x).is_close(d, -d)
            }
        }
        lt_imp_minus_pos(a, b)
        (b - a).is_positive
        eps_smaller_than_both(delta, b - a)
        let h0: Real satisfy {
            h0.is_positive and h0 < delta and h0 < b - a
        }
        let x = a + h0
        lt_add_pos(a, h0)
        a < x
        // x < b
        lt_add_right(h0, b - a, a)
        h0 + a < (b - a) + a
        (b - a) + a = b + -a + a
        b + -a + a = b + (-a + a)
        add_neg_eq_zero(a)
        -a + a = Real.0
        b + Real.0 = b
        (b - a) + a = b
        h0 + a < b
        add_comm(h0, a)
        h0 + a = a + h0
        x < b
        // x != a and x close to a within delta
        lt_imp_ne_symm(a, x)
        x != a
        x - a = (a + h0) - a
        add_comm(a, h0)
        a + h0 = h0 + a
        (a + h0) - a = (h0 + a) - a
        sub_cancels(h0, a)
        h0 + a - a = h0
        x - a = h0
        pos_imp_eq_abs(h0)
        h0 = h0.abs
        (x - a).abs = h0.abs
        (x - a).abs = h0
        h0 < delta
        (x - a).abs < delta
        x.is_close(a, delta)
        x != a and x.is_close(a, delta)
        forall_elim[Real](function(y: Real) {
            y != a and y.is_close(a, delta) implies
                difference_quotient(h, a, y).is_close(d, -d)
        }, x)
        function(y: Real) {
            y != a and y.is_close(a, delta) implies
                difference_quotient(h, a, y).is_close(d, -d)
        }(x)
        difference_quotient(h, a, x).is_close(d, -d)
        close_imp_bounds(difference_quotient(h, a, x), d, -d)
        difference_quotient(h, a, x) < d + -d
        add_neg_eq_zero(d)
        d + -d = Real.0
        difference_quotient(h, a, x) < Real.0
        difference_quotient(h, a, x) = (h(x) - h(a)) / (x - a)
        x - a = h0
        difference_quotient(h, a, x) = (h(x) - h(a)) / h0
        (h(x) - h(a)) / h0 < Real.0
        pos_gt_zero(h0)
        h0 > Real.0
        div_lt_mul_pos(h(x) - h(a), h0, Real.0)
        h(x) - h(a) < h0 * Real.0
        mul_zero_right(h0)
        h0 * Real.0 = Real.0
        h(x) - h(a) < Real.0
        lt_add_right(h(x) - h(a), Real.0, h(a))
        (h(x) - h(a)) + h(a) < Real.0 + h(a)
        add_zero_left(h(a))
        Real.0 + h(a) = h(a)
        h(x) - h(a) + h(a) = h(x) + -h(a) + h(a)
        add_comm(-h(a), h(a))
        -h(a) + h(a) = h(a) + -h(a)
        add_neg_eq_zero(h(a))
        h(a) + -h(a) = Real.0
        h(x) + (-h(a) + h(a)) = h(x) + Real.0
        add_zero_right(h(x))
        h(x) + Real.0 = h(x)
        h(x) - h(a) + h(a) = h(x)
        h(x) < h(a)
        exists(z: Real) { a < z and z < b and h(z) < h(a) }
    }
}

/// A positive derivative at `b` forces the function strictly below its
/// value at `b` just to the left of `b`.
theorem derivative_positive_imp_dips_left(h: Real -> Real, a: Real, b: Real, d: Real) {
    a < b and has_derivative_at(h, b, d) and Real.0 < d
    implies exists(x: Real) { a < x and x < b and h(x) < h(b) }
} by {
    if a < b and has_derivative_at(h, b, d) and Real.0 < d {
        gt_zero_imp_pos(d)
        d.is_positive
        has_derivative_at_delta(h, b, d, d)
        let delta: Real satisfy {
            delta.is_positive and forall(x: Real) {
                x != b and x.is_close(b, delta)
                implies difference_quotient(h, b, x).is_close(d, d)
            }
        }
        lt_imp_minus_pos(a, b)
        (b - a).is_positive
        eps_smaller_than_both(delta, b - a)
        let h0: Real satisfy {
            h0.is_positive and h0 < delta and h0 < b - a
        }
        let x = b - h0
        // x < b
        h0.is_positive
        neg_pos_is_neg(h0)
        (-h0).is_negative
        neg_lt_zero(-h0)
        -h0 < Real.0
        lt_add_right(-h0, Real.0, b)
        -h0 + b < Real.0 + b
        add_zero_left(b)
        Real.0 + b = b
        -h0 + b < b
        b - h0 = b + -h0
        b + -h0 < b
        x < b
        // a < x
        lt_add_right(h0, b - a, a)
        h0 + a < (b - a) + a
        (b - a) + a = b + -a + a
        b + -a + a = b + (-a + a)
        add_neg_eq_zero(a)
        -a + a = Real.0
        b + Real.0 = b
        (b - a) + a = b
        h0 + a < b
        lt_add_right(h0 + a, b, -h0)
        (h0 + a) + -h0 < b + -h0
        h0 + a - h0 = a + h0 - h0
        add_comm(h0, a)
        h0 + a = a + h0
        a + h0 - h0 = a + (h0 + -h0)
        add_neg_eq_zero(h0)
        h0 + -h0 = Real.0
        a + Real.0 = a
        a + h0 - h0 = a
        (h0 + a) + -h0 = a
        b - h0 = b + -h0
        a < b - h0
        a < x
        // x != b and x close to b within delta
        lt_imp_ne_symm(x, b)
        x != b
        x - b = (b - h0) - b
        b - h0 = b + -h0
        (b + -h0) - b = b + -h0 + -b
        add_comm(b, -h0)
        b + -h0 = -h0 + b
        (-h0 + b) - b = -h0 + b + -b
        add_comm(b, -b)
        b + -b = -b + b
        add_neg_eq_zero(b)
        b + -b = Real.0
        -h0 + (b + -b) = -h0 + Real.0
        add_zero_right(-h0)
        -h0 + Real.0 = -h0
        (-h0 + b) - b = -h0
        x - b = -h0
        abs_neg(h0)
        (-h0).abs = h0.abs
        pos_imp_eq_abs(h0)
        h0 = h0.abs
        (x - b).abs = (-h0).abs
        (x - b).abs = h0
        h0 < delta
        (x - b).abs < delta
        x.is_close(b, delta)
        x != b and x.is_close(b, delta)
        forall_elim[Real](function(y: Real) {
            y != b and y.is_close(b, delta) implies
                difference_quotient(h, b, y).is_close(d, d)
        }, x)
        function(y: Real) {
            y != b and y.is_close(b, delta) implies
                difference_quotient(h, b, y).is_close(d, d)
        }(x)
        difference_quotient(h, b, x).is_close(d, d)
        close_imp_bounds(difference_quotient(h, b, x), d, d)
        difference_quotient(h, b, x) > d - d
        d - d = d + -d
        add_neg_eq_zero(d)
        d + -d = Real.0
        difference_quotient(h, b, x) > Real.0
        // difference_quotient(h, b, x) = (h(x) - h(b)) / (x - b) = (h(x) - h(b)) / (-h0)
        difference_quotient(h, b, x) = (h(x) - h(b)) / (x - b)
        x - b = -h0
        difference_quotient(h, b, x) = (h(x) - h(b)) / (-h0)
        (h(x) - h(b)) / (-h0) > Real.0
        // (h(x) - h(b)) / (-h0) = (h(b) - h(x)) / h0
        h0.is_positive
        lt_imp_ne(Real.0, h0)
        Real.0 < h0
        Real.0 != h0
        h0 != Real.0
        div_neg_denominator(h(x) - h(b), h0)
        (h(x) - h(b)) / (-h0) = -((h(x) - h(b)) / h0)
        neg_div(h(x) - h(b), h0)
        (-(h(x) - h(b))) / h0 = -((h(x) - h(b)) / h0)
        h(x) - h(b) = h(x) + -h(b)
        neg_distrib(h(x), -h(b))
        -(h(x) + -h(b)) = -h(x) + -(-h(b))
        neg_neg(h(b))
        -(-h(b)) = h(b)
        -(h(x) + -h(b)) = -h(x) + h(b)
        h(b) - h(x) = h(b) + -h(x)
        -h(x) + h(b) = h(b) - h(x)
        -(h(x) - h(b)) = h(b) - h(x)
        (h(b) - h(x)) / h0 = -((h(x) - h(b)) / h0)
        (h(x) - h(b)) / (-h0) = (h(b) - h(x)) / h0
        (h(b) - h(x)) / h0 > Real.0
        // -(h(b) - h(x)) / h0 < 0, and h0 > 0, so -(h(b) - h(x)) < 0
        gt_zero_imp_pos((h(b) - h(x)) / h0)
        ((h(b) - h(x)) / h0).is_positive
        neg_pos_is_neg((h(b) - h(x)) / h0)
        (-((h(b) - h(x)) / h0)).is_negative
        neg_lt_zero(-((h(b) - h(x)) / h0))
        -((h(b) - h(x)) / h0) < Real.0
        neg_div(h(b) - h(x), h0)
        (-(h(b) - h(x))) / h0 = -((h(b) - h(x)) / h0)
        (-(h(b) - h(x))) / h0 < Real.0
        pos_gt_zero(h0)
        h0 > Real.0
        div_lt_mul_pos(-(h(b) - h(x)), h0, Real.0)
        -(h(b) - h(x)) < h0 * Real.0
        mul_zero_right(h0)
        h0 * Real.0 = Real.0
        -(h(b) - h(x)) < Real.0
        // -(h(b) - h(x)) < 0 gives 0 < h(b) - h(x), hence h(x) < h(b)
        h(b) - h(x) = h(b) + -h(x)
        neg_distrib(h(b), -h(x))
        -(h(b) + -h(x)) = -h(b) + -(-h(x))
        neg_neg(h(x))
        -(-h(x)) = h(x)
        -(h(b) + -h(x)) = -h(b) + h(x)
        -(h(b) - h(x)) = -(h(b) + -h(x))
        -(h(b) - h(x)) = -h(b) + h(x)
        -h(b) + h(x) + (h(b) - h(x)) = h(x) + -h(b) + (h(b) - h(x))
        h(b) - h(x) = h(b) + -h(x)
        h(x) + -h(b) + (h(b) + -h(x)) = h(x) + (-h(b) + h(b)) + -h(x)
        add_neg_eq_zero(h(b))
        h(b) + -h(b) = Real.0
        h(x) + Real.0 + -h(x) = h(x) + -h(x)
        add_neg_eq_zero(h(x))
        h(x) + -h(x) = Real.0
        -(h(b) - h(x)) + (h(b) - h(x)) = Real.0
        lt_add_right(-(h(b) - h(x)), Real.0, h(b) - h(x))
        -(h(b) - h(x)) + (h(b) - h(x)) < Real.0 + (h(b) - h(x))
        add_zero_left(h(b) - h(x))
        Real.0 + (h(b) - h(x)) = h(b) - h(x)
        Real.0 < h(b) - h(x)
        lt_add_right(Real.0, h(b) - h(x), h(x))
        Real.0 + h(x) < (h(b) - h(x)) + h(x)
        add_zero_left(h(x))
        Real.0 + h(x) = h(x)
        h(b) - h(x) + h(x) = h(b)
        h(x) < h(b)
        exists(z: Real) { a < z and z < b and h(z) < h(b) }
    }
}

/// Darboux's theorem: derivatives have the intermediate value property.
///
/// If `f` is differentiable at every point and `df(a) < t < df(b)`, then
/// some `c` strictly between `a` and `b` has `df(c) = t`.
theorem darboux_theorem(f: Real -> Real, df: Real -> Real, a: Real, b: Real, t: Real) {
    a < b and
    (forall(x: Real) { has_derivative_at(f, x, df(x)) }) and
    df(a) < t and t < df(b)
    implies exists(c: Real) {
        a < c and c < b and df(c) = t
    }
} by {
    if a < b and (forall(x: Real) { has_derivative_at(f, x, df(x)) }) and
       df(a) < t and t < df(b) {
        let g = darboux_aux(f, t)
        // g is continuous on [a, b], so it attains a minimum there.
        darboux_aux_continuous_on_closed(f, df, t, a, b)
        continuous_on_closed(g, a, b)
        continuous_closed_interval_attains_minimum(g, a, b)
        let x0: Real satisfy {
            closed_interval_set(a, b).contains(x0) and
            (forall(y: Real) { closed_interval_set(a, b).contains(y) implies g(x0) <= g(y) })
        }
        closed_interval_set_lower_le(a, b, x0)
        a <= x0
        closed_interval_set_le_upper(a, b, x0)
        x0 <= b
        // g'(a) = df(a) - t < 0, so g dips below g(a) to the right of a;
        // the minimum is therefore not at a.
        has_derivative_at(f, a, df(a))
        darboux_aux_has_derivative_at(f, df, t, a)
        has_derivative_at(g, a, df(a) - t)
        df(a) < t
        lt_add_right(df(a), t, -t)
        df(a) + -t < t + -t
        add_neg_eq_zero(t)
        t + -t = Real.0
        df(a) - t = df(a) + -t
        df(a) - t < Real.0
        derivative_negative_imp_dips_right(g, a, b, df(a) - t)
        exists(x: Real) { a < x and x < b and g(x) < g(a) }
        let xl: Real satisfy { a < xl and xl < b and g(xl) < g(a) }
        a < xl
        xl < b
        g(xl) < g(a)
        if x0 = a {
            lt_imp_lte(a, xl)
            a <= xl
            lt_imp_lte(xl, b)
            xl <= b
            closed_interval(a, b, xl)
            closed_interval_set_contains_eq(a, b, xl)
            closed_interval_set(a, b).contains(xl)
            g(x0) <= g(xl)
            x0 = a
            g(a) <= g(xl)
            lte_imp_not_lt(g(a), g(xl))
            not g(xl) < g(a)
            g(xl) < g(a)
            false
        }
        x0 != a
        if not a < x0 {
            not_lt_imp_gte(a, x0)
            a >= x0
            x0 <= a
            lte_antisymm(a, x0)
            a = x0
            x0 != a
            false
        }
        a < x0
        // g'(b) = df(b) - t > 0, so g dips below g(b) to the left of b;
        // the minimum is therefore not at b.
        has_derivative_at(f, b, df(b))
        darboux_aux_has_derivative_at(f, df, t, b)
        has_derivative_at(g, b, df(b) - t)
        t < df(b)
        lt_add_right(t, df(b), -t)
        t + -t < df(b) + -t
        add_neg_eq_zero(t)
        t + -t = Real.0
        df(b) - t = df(b) + -t
        Real.0 < df(b) - t
        derivative_positive_imp_dips_left(g, a, b, df(b) - t)
        exists(x: Real) { a < x and x < b and g(x) < g(b) }
        let xr: Real satisfy { a < xr and xr < b and g(xr) < g(b) }
        g(xr) < g(b)
        if x0 = b {
            lt_imp_lte(a, xr)
            a <= xr
            lt_imp_lte(xr, b)
            xr <= b
            closed_interval(a, b, xr)
            closed_interval_set_contains_eq(a, b, xr)
            closed_interval_set(a, b).contains(xr)
            g(x0) <= g(xr)
            x0 = b
            g(b) <= g(xr)
            lte_imp_not_lt(g(b), g(xr))
            not g(xr) < g(b)
            g(xr) < g(b)
            false
        }
        x0 != b
        if not x0 < b {
            not_lt_imp_gte(x0, b)
            x0 >= b
            b <= x0
            lte_antisymm(x0, b)
            x0 = b
            x0 != b
            false
        }
        x0 < b
        // Fermat's theorem on interior extrema applied to -g.
        has_derivative_at(f, x0, df(x0))
        darboux_aux_has_derivative_at(f, df, t, x0)
        has_derivative_at(g, x0, df(x0) - t)
        derivative_pointwise_neg(g, x0, df(x0) - t)
        has_derivative_at(pointwise_neg(g), x0, -(df(x0) - t))
        forall(y: Real) {
            if a < y and y < b {
                lt_imp_lte(a, y)
                a <= y
                lt_imp_lte(y, b)
                y <= b
                closed_interval(a, b, y)
                closed_interval_set_contains_eq(a, b, y)
                closed_interval_set(a, b).contains(y)
                g(x0) <= g(y)
                lte_imp_neg_lte_neg(g(x0), g(y))
                -g(y) <= -g(x0)
                pointwise_neg(g, y) = -g(y)
                pointwise_neg(g, x0) = -g(x0)
                pointwise_neg(g, y) <= pointwise_neg(g, x0)
            }
        }
        fermat_interior_maximum(pointwise_neg(g), a, b, x0, -(df(x0) - t))
        -(df(x0) - t) = Real.0
        -(-(df(x0) - t)) = -Real.0
        neg_neg(df(x0) - t)
        -(-(df(x0) - t)) = df(x0) - t
        neg_zero
        -Real.0 = Real.0
        df(x0) - t = Real.0
        sub_zero_imp_eq(df(x0), t)
        df(x0) = t
        exists(c: Real) { a < c and c < b and df(c) = t }
    }
}
