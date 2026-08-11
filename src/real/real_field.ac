from algebra.inverse import Inverse
from list import partial
from nat import Nat, lte_trans
from rat import Rat
from algebra.monoid.monoid import Monoid
from real.real_seq import cauchy_bound, tail_bound, neg_rat_seq, eventual_eq
from real.real_ring import lift_seq, converges, converges_to, limit, eventual_lb, limit_rat, rat_seq, mul_rat_seq
from real.real_series import Real, neg_seq
numerals Real

// This file defines real division and proves theorems about it.

define recip_rat_seq(a: Nat -> Rat, n: Nat) -> Rat {
    a(n).inverse
}

theorem neg_recip_rat_seq(a: Nat -> Rat) {
    neg_rat_seq(recip_rat_seq(a)) = recip_rat_seq(neg_rat_seq(a))
} by {
    forall(n: Nat) {
        neg_rat_seq(a, n) = -a(n)
        neg_rat_seq(recip_rat_seq(a), n) = -recip_rat_seq(a, n)
        recip_rat_seq(a, n) = a(n).inverse
        recip_rat_seq(neg_rat_seq(a), n) = neg_rat_seq(a, n).inverse
        neg_rat_seq(a, n).inverse = (-a(n)).inverse
        -Rat.1 * a(n).inverse = -a(n).inverse
        Rat.1 * (-a(n)).inverse = (-a(n)).inverse
        -Rat.1 / -neg_rat_seq(a, n) = Rat.1 / neg_rat_seq(a, n)
        neg_rat_seq(recip_rat_seq(a), n) = recip_rat_seq(neg_rat_seq(a), n)
    }
}

// Division works for positive reals
theorem recip_rat_seq_pos_converges(a: Nat -> Rat, b: Real) {
    converges_to(lift_seq(a), b) and b.is_positive
    implies converges(lift_seq(recip_rat_seq(a)))
} by {
    from real.real_base import rat_between_reals
    let r0: Rat satisfy {
        Real.0 < Real.from_rat(r0) and Real.from_rat(r0) < b
    }
    Real.from_rat(Rat.0) < Real.from_rat(r0)
    Rat.0 < r0
    exists(b_lb: Rat) {
        Rat.0 < b_lb and Real.from_rat(b_lb) < b
    }
    let b_lb: Rat satisfy {
        Rat.0 < b_lb and Real.from_rat(b_lb) < b
    }
    let lsa = lift_seq(a)
    from real.real_seq import lt_converges_to_imp_lb
    eventual_lb(lsa, Real.from_rat(b_lb))
    let n_lb: Nat satisfy {
        forall(i: Nat) {
            n_lb <= i implies Real.from_rat(b_lb) <= lsa(i)
        }
    }
    forall(i: Nat) {
        if n_lb < i {
            n_lb <= i
            Real.from_rat(b_lb) <= lsa(i)
        }
    }
    exists(n1: Nat) {
        forall(i: Nat) {
            n1 < i implies Real.from_rat(b_lb) <= lsa(i)
        }
    }
    let n1: Nat satisfy {
        forall(i: Nat) {
            n1 < i implies Real.from_rat(b_lb) <= lsa(i)
        }
    }
    forall(i: Nat) {
        if n1 < i {
            b_lb <= a(i)
        }
    }

    forall(eps: Real) {
        if eps.is_positive {
            let reps: Rat satisfy {
                reps.is_positive and Real.from_rat(reps) < eps
            }

            // We had to work backwards to find this condition
            (reps * (b_lb * b_lb)).is_positive
            let eps2: Rat satisfy {
                eps2.is_positive and eps2 < reps * (b_lb * b_lb)
            }
            let eps3: Rat satisfy {
                eps3 + eps3 = eps2
            }
            eps3.is_positive

            // Find where the original sequence is within eps_bound of b
            let n2: Nat satisfy {
                tail_bound(lift_seq(a), b, n2, Real.from_rat(eps3))
            }

            // Choose a bound above both
            let n_bound = n1.max(n2).suc
            n1 <= n1.max(n2)
            n2 <= n1.max(n2)
            n1 < n_bound
            n2 < n_bound
            exists(n: Nat) {
                n1 < n and n2 < n
            }
            let n: Nat satisfy {
                n1 < n and n2 < n
            }

            // Now we show that the inverse sequence satisfies the Cauchy property
            forall(i: Nat, j: Nat) {
                if n <= i and n <= j {
                    // Both terms are close to b
                    tail_bound(lift_seq(a), b, n2, Real.from_rat(eps3))
                    n2 < n
                    n2 <= i
                    lift_seq(a)(i).is_close(b, Real.from_rat(eps3))
                    n2 <= j
                    lift_seq(a)(j).is_close(b, Real.from_rat(eps3))

                    // So their diff is small
                    lift_seq(a, i).is_close(lift_seq(a, j), Real.from_rat(eps3) + Real.from_rat(eps3))
                    a(i).is_close(a(j), eps2)
                    let diff = a(j) - a(i)

                    // Both terms are non-zero
                    b_lb <= a(i)
                    b_lb <= a(j)
                    a(j) != Rat.0

                    // So their product is large
                    let prod = a(i) * a(j)
                    b_lb * b_lb <= prod
                    b_lb * b_lb <= prod.abs

                    diff.abs * (b_lb * b_lb) < eps2 * prod.abs
                    diff.abs < (eps2 / (b_lb * b_lb)) * prod.abs
                    (diff / prod).abs < eps2 / (b_lb * b_lb)
                    (diff / prod).abs < reps
                    a(i).inverse - a(j).inverse = ((a(j) - a(i)) / (a(i) * a(j)))
                    a(i).inverse = recip_rat_seq(a, i)
                    a(j).inverse = recip_rat_seq(a, j)
                    recip_rat_seq(a, i).is_close(recip_rat_seq(a, j), reps)
                    Real.from_rat(recip_rat_seq(a, i)).is_close(Real.from_rat(recip_rat_seq(a, j)), Real.from_rat(reps))
                    Real.from_rat(recip_rat_seq(a, i)) = lift_seq(recip_rat_seq(a), i)
                    Real.from_rat(recip_rat_seq(a, j)) = lift_seq(recip_rat_seq(a), j)
                    lift_seq(recip_rat_seq(a))(i).is_close(lift_seq(recip_rat_seq(a))(j), Real.from_rat(reps))
                    lift_seq(recip_rat_seq(a))(i).is_close(lift_seq(recip_rat_seq(a))(j), eps)
                }
            }

            // This establishes the Cauchy criterion
            exists(k0: Nat, k1: Nat) {
                n <= k0 and n <= k1 and
                not lift_seq(recip_rat_seq(a))(k0).is_close(lift_seq(recip_rat_seq(a))(k1), eps)
            } or cauchy_bound(lift_seq(recip_rat_seq(a)), n, eps)
            cauchy_bound(lift_seq(recip_rat_seq(a)), n, eps)
        }
    }

    // Therefore the sequence converges
}

// Division works for all nonzero reals
theorem recip_rat_seq_converges(a: Nat -> Rat, b: Real) {
    converges_to(lift_seq(a), b) and b != Real.0
    implies converges(lift_seq(recip_rat_seq(a)))
} by {
    if b.is_positive {
        // This is the previous theorem
    } else {
        b.is_negative
        (-b).is_positive
        limit(lift_seq(a)) = b
        converges(lift_seq(a))
        converges_to(lift_seq(neg_rat_seq(a)), -limit_rat(a))
        limit_rat(a) = b
        converges_to(lift_seq(neg_rat_seq(a)), -b)
        converges(lift_seq(recip_rat_seq(neg_rat_seq(a))))
        recip_rat_seq(neg_rat_seq(a)) = neg_rat_seq(recip_rat_seq(a))
        converges(lift_seq(neg_rat_seq(recip_rat_seq(a))))
        lift_seq(neg_rat_seq(recip_rat_seq(a))) = neg_seq(lift_seq(recip_rat_seq(a)))
        converges(lift_seq(recip_rat_seq(a)))
    }
}

/// The inverse of this real number (`1/x`). For zero, returns zero.
instance Real: Inverse {
    define inverse(self) -> Real {
        if self = Real.0 {
            Real.0
        } else {
            limit_rat(recip_rat_seq(rat_seq(self)))
        }
    }
}

define eventually_nonzero(a: Nat -> Real) -> Bool {
    exists(n: Nat) {
        forall(i: Nat) {
            n <= i implies a(i) != Real.0
        }
    }
}

theorem pos_imp_eventually_nonzero(a: Nat -> Real) {
    converges(a) and limit(a).is_positive
    implies eventually_nonzero(a)
} by {
    if converges(a) and limit(a).is_positive {
        converges_to(a, limit(a))
        exists(n: Nat) {
            tail_bound(a, limit(a), n, limit(a))
        }
        exists(n: Nat) {
            forall(i: Nat) {
                n <= i implies a(i).is_close(limit(a), limit(a))
            }
        }
        let n: Nat satisfy {
            forall(i: Nat) {
                n <= i implies a(i).is_close(limit(a), limit(a))
            }
        }
        forall(i: Nat) {
            if n <= i {
                a(i).is_close(limit(a), limit(a))
                Real.0.is_close(limit(a), limit(a)) implies limit(a) < Real.0 + limit(a)
                Real.0 + limit(a) = limit(a)
                a(i) != Real.0
            }
        }
        eventually_nonzero(a)
    }
}

theorem nonzero_imp_eventually_nonzero(a: Nat -> Real) {
    converges(a) and limit(a) != Real.0
    implies eventually_nonzero(a)
} by {
    if limit(a).is_positive {
    } else {
        limit(a).is_negative
        (-limit(a)).is_positive
        converges(neg_seq(a))
        limit(neg_seq(a)) = -limit(a)
        limit(neg_seq(a)).is_positive
        eventually_nonzero(neg_seq(a))
        let n: Nat satisfy {
            forall(i: Nat) {
                n <= i implies neg_seq(a)(i) != Real.0
            }
        }
        forall(i: Nat) {
            if n <= i {
                a(i) != Real.0
            }
        }
        eventually_nonzero(a)
    }
}

theorem mul_inverse(a: Real) {
    a != Real.0 implies a * a.inverse = Real.1
} by {
    from real.real_seq import rat_seq_converges_to, converges_to_imp_converges, converges_to_unique
    converges(lift_seq(rat_seq(a)))
    limit(lift_seq(rat_seq(a))) = a
    limit(lift_seq(rat_seq(a))) != Real.0
    exists(n: Nat) {
        forall(i: Nat) {
            n <= i implies lift_seq(rat_seq(a))(i) != Real.0
        }
    }
    let n: Nat satisfy {
        forall(i: Nat) {
            n <= i implies lift_seq(rat_seq(a))(i) != Real.0
        }
    }

    // We want to show that the product of the sequences is eventually
    // just a constant one.
    let prod = mul_rat_seq(rat_seq(a), recip_rat_seq(rat_seq(a)))

    forall(i: Nat) {
        if n <= i {
            lift_seq(rat_seq(a))(i) != Real.0
            lift_seq(rat_seq(a), i) = Real.from_rat(rat_seq(a, i))
            Real.from_rat(rat_seq(a, i)) != Real.0
            rat_seq(a, i) != Rat.0
            recip_rat_seq(rat_seq(a), i) = rat_seq(a, i).inverse
            rat_seq(a, i) * rat_seq(a, i).inverse = Rat.1
            rat_seq(a, i) * recip_rat_seq(rat_seq(a), i) = Rat.1
            lift_seq(prod)(i) = Real.1
        }
    }
    exists(n0: Nat) {
        n0 = n and forall(i: Nat) {
            n0 <= i implies lift_seq(prod)(i) = Real.1
        }
    }
    exists(n0: Nat) {
        forall(i: Nat) {
            n0 <= i implies lift_seq(prod)(i) = Real.1
        }
    }
    eventual_eq(lift_seq(prod), Real.1)

    limit_rat(rat_seq(a)) * limit_rat(recip_rat_seq(rat_seq(a))) = limit_rat(mul_rat_seq(rat_seq(a), recip_rat_seq(rat_seq(a))))
}

theorem zero_inverse {
    Real.0.inverse = Real.0
}

from algebra.field.field import Field

theorem zero_is_different_than_one {
    Real.0 != Real.1
}

instance Real: Field

from algebra.field.field import mul_not_zero

theorem real_no_zero_divisors(a: Real, b: Real) {
    a * b = Real.0 implies a = Real.0 or b = Real.0
} by {
    if a != Real.0 and b != Real.0 {
        mul_not_zero[Real](a, b)
        a * b != Real.0
    }
}

from algebra.integral_domain import IntegralDomain

instance Real: IntegralDomain

from ordered_field import OrderedField, mul_le_mul_of_nonneg_right, mul_le_mul_of_nonneg_left,
    mul_lt_mul_of_pos_right

instance Real: OrderedField

attributes Real {
    /// The quotient of two real numbers (`self/other`).
    define div(self, other: Real) -> Real {
        self * other.inverse
    }
}

theorem mul_left_cancel(a: Real, b: Real, c: Real) {
    a = b * c and b != Real.0 implies a / b = c
} by {
    if a = b * c and b != Real.0 {
        a * b.inverse = a / b
        b * b.inverse = Real.1
        b.inverse * (b * c) = b.inverse * b * c
        b.inverse * b = b * b.inverse
        b.inverse * a = a * b.inverse
        a / b = c
    }
}

/// If a * b <= c and b > 0, then a <= c / b.
theorem div_le_of_mul_le(a: Real, b: Real, c: Real) {
    a * b <= c and b > Real.0
    implies
    a <= c / b
} by {
    b != Real.0
    // We have a * b <= c
    // Multiply both sides by b.inverse
    // Since b > 0, we have b.inverse > 0
    // So multiplication preserves <=
    a * b * b.inverse <= c * b.inverse
    a * (b * b.inverse) <= c * b.inverse
    b * b.inverse = Real.1
    a * Real.1 <= c * b.inverse
    a <= c * b.inverse
    c / b = c * b.inverse
    a <= c / b
}

/// If a <= b and c > 0, then a * c <= b * c.
theorem mul_le_mul_pos_right(a: Real, b: Real, c: Real) {
    a <= b and c > Real.0
    implies
    a * c <= b * c
} by {
    if a <= b and c > Real.0 {
        Real.0 <= c
        mul_le_mul_of_nonneg_right(a, b, c)
        a * c <= b * c
    }
}

/// If a <= b and c > 0, then c * a <= c * b.
theorem mul_le_mul_pos_left(a: Real, b: Real, c: Real) {
    a <= b and c > Real.0
    implies
    c * a <= c * b
} by {
    if a <= b and c > Real.0 {
        Real.0 <= c
        mul_le_mul_of_nonneg_left(a, b, c)
        c * a <= c * b
    }
}

/// If a / b < c and b > 0, then a < b * c.
theorem div_lt_mul_pos(a: Real, b: Real, c: Real) {
    a / b < c and b > Real.0
    implies
    a < b * c
} by {
    if a / b < c and b > Real.0 {
        b != Real.0
        // We have a / b < c, which means a * b.inverse < c
        a / b = a * b.inverse
        a * b.inverse < c

        // Multiply both sides by b, which is positive.
        mul_lt_mul_of_pos_right(a * b.inverse, c, b)
        (a * b.inverse) * b < c * b

        // Simplify left side
        a * (b.inverse * b) < c * b
        b.inverse * b = Real.1
        a * Real.1 < c * b
        a < c * b
        a < b * c
    }
}

/// If c > 0, then c * (b / c) = b.
theorem mul_div_cancel(b: Real, c: Real) {
    c != Real.0
    implies
    c * (b / c) = b
} by {
    c * (b / c) = c * (b * c.inverse)
    c * (b * c.inverse) = c * c.inverse * b
    c * c.inverse = Real.1
    Real.1 * b = b
}

/// Multiplication with inverse commutes: a * (b * c) = b * (a * c).
theorem mul_inverse_comm(a: Real, b: Real, c: Real) {
    a * (b * c) = b * (a * c)
}

/// Division by product: a / (b * c) = (a / b) / c when b, c != 0.
/// This follows from inverse_dist and associativity.
// theorem div_mul_assoc(a: Real, b: Real, c: Real) {
//     b != Real.0 and c != Real.0
//     implies
//     a / (b * c) = (a / b) / c
// } by {
//     if b != Real.0 and c != Real.0 {
//         b * c != Real.0
//         a / (b * c) = a * (b * c).inverse
//         (b * c).inverse = b.inverse * c.inverse
//         a / (b * c) = a * (b.inverse * c.inverse)
//
//         // Show (a / b) / c = (a * b.inverse) * c.inverse
//         (a / b) / c = (a * b.inverse) / c
//         (a * b.inverse) / c = (a * b.inverse) * c.inverse
//
//         // These are equal by associativity
//         a * (b.inverse * c.inverse) = (a * b.inverse) * c.inverse
//     }
// }

/// Cancellation: a / (b * a) = 1 / b when a, b != 0.
theorem div_mul_cancel_right(a: Real, b: Real) {
    a != Real.0 and b != Real.0
    implies
    a / (b * a) = Real.1 / b
} by {
    if a != Real.0 and b != Real.0 {
        // Use commutativity: b * a = a * b
        b * a = a * b
        a / (b * a) = a / (a * b)

        // Expand a / (a * b)
        a * b != Real.0
        a / (a * b) = a * (a * b).inverse
        (a * b).inverse = a.inverse * b.inverse
        a / (a * b) = a * (a.inverse * b.inverse)

        // Rearrange using associativity and commutativity
        a * (a.inverse * b.inverse) = (a * a.inverse) * b.inverse
        a * a.inverse = Real.1
        (a * a.inverse) * b.inverse = Real.1 * b.inverse
        Real.1 * b.inverse = b.inverse
        b.inverse = Real.1 / b

        a / (b * a) = Real.1 / b
    }
}

/// Cancellation (left): (a * b) / a = b when a != 0.
theorem div_mul_cancel_left(a: Real, b: Real) {
    a != Real.0
    implies
    (a * b) / a = b
} by {
    if a != Real.0 {
        (a * b) / a = (a * b) * a.inverse
        (a * b) * a.inverse = a * (b * a.inverse)
        a * (b * a.inverse) = a * (a.inverse * b)
        a * (a.inverse * b) = (a * a.inverse) * b
        a * a.inverse = Real.1
        Real.1 * b = b
    }
}

/// Bilateral cancellation: (a * b) / (c * b) = a / c when b, c != 0.
theorem div_cancel_common(a: Real, b: Real, c: Real) {
    b != Real.0 and c != Real.0
    implies
    (a * b) / (c * b) = a / c
} by {
    if b != Real.0 and c != Real.0 {
        // Show (c * b) * (a / c) = a * b, then apply mul_left_cancel.
        c * b != Real.0
        (c * b) * (a / c) = (c * b) * (a * c.inverse)
        (c * b) * (a * c.inverse) = c * (b * (a * c.inverse))
        c * (b * (a * c.inverse)) = c * (a * (b * c.inverse))
        c * (a * (b * c.inverse)) = c * (a * (c.inverse * b))
        c * (a * (c.inverse * b)) = c * ((a * c.inverse) * b)
        c * ((a * c.inverse) * b) = (c * (a * c.inverse)) * b
        (c * (a * c.inverse)) * b = ((c * a) * c.inverse) * b
        ((c * a) * c.inverse) * b = ((a * c) * c.inverse) * b
        ((a * c) * c.inverse) * b = (a * (c * c.inverse)) * b
        (a * (c * c.inverse)) * b = (a * Real.1) * b
        (a * Real.1) * b = a * b
        (c * b) * (a / c) = a * b
        (a * b) / (c * b) = a / c
    }
}

/// Convert product equality to division equality.
/// If a * b = c * d, then a / d = c / b (when b, d != 0).
theorem prod_eq_to_div_eq(a: Real, b: Real, c: Real, d: Real) {
    a * b = c * d and b != Real.0 and d != Real.0
    implies
    a / d = c / b
} by {
    if a * b = c * d and b != Real.0 and d != Real.0 {
        // Multiply both sides of a * b = c * d by 1/(b * d)
        // (a * b) / (b * d) = (c * d) / (b * d)
        b * d != Real.0
        (a * b) / (b * d) = (c * d) / (b * d)

        // Apply div_cancel_common to both sides, expanded explicitly.
        (a * b) / (d * b) = a / d
        d * b = b * d
        (a * b) / (b * d) = a / d
        (c * d) / (b * d) = c / b

        a / d = c / b
    }
}

/// Reciprocal of a division: (a/b).inverse = b/a when a, b != 0.
theorem inverse_div(a: Real, b: Real) {
    a != Real.0 and b != Real.0
    implies
    (a / b).inverse = b / a
} by {
    if a != Real.0 and b != Real.0 {
        // a/b != 0
        a / b != Real.0

        // (a/b) * (a/b).inverse = 1
        (a / b) * (a / b).inverse = Real.1

        // (a/b) * (b/a) = ?
        // = (a * b.inverse) * (b * a.inverse)
        // = a * (b.inverse * (b * a.inverse))
        // = a * ((b.inverse * b) * a.inverse)
        // = a * (1 * a.inverse)
        // = a * a.inverse
        // = 1

        (a / b) * (b / a) = (a * b.inverse) * (b * a.inverse)
        (a * b.inverse) * (b * a.inverse) = a * (b.inverse * (b * a.inverse))
        a * (b.inverse * (b * a.inverse)) = a * ((b.inverse * b) * a.inverse)
        b.inverse * b = Real.1
        a * ((b.inverse * b) * a.inverse) = a * (Real.1 * a.inverse)
        a * (Real.1 * a.inverse) = a * a.inverse
        a * a.inverse = Real.1

        (a / b) * (b / a) = Real.1

        // Since both (a/b).inverse and b/a multiply with a/b to give 1,
        // they must be equal
        (a / b).inverse = b / a
    }
}

/// Multiplication of fractions: (a/b) * (c/d) = (a*c) / (b*d) when b, d != 0.
theorem mul_div(a: Real, b: Real, c: Real, d: Real) {
    b != Real.0 and d != Real.0
    implies
    (a / b) * (c / d) = (a * c) / (b * d)
} by {
    if b != Real.0 and d != Real.0 {
        // Division definitions
        a / b = a * b.inverse
        c / d = c * d.inverse
        (a * c) / (b * d) = (a * c) * (b * d).inverse

        // Expand LHS
        (a / b) * (c / d) = (a * b.inverse) * (c * d.inverse)
        (a * b.inverse) * (c * d.inverse) = a * (b.inverse * c * d.inverse)
        a * (b.inverse * c * d.inverse) = a * (c * b.inverse * d.inverse)
        a * (c * b.inverse * d.inverse) = a * c * (b.inverse * d.inverse)

        // Use inverse_dist
        b * d != Real.0
        b.inverse * d.inverse = (b * d).inverse
        a * c * (b.inverse * d.inverse) = a * c * (b * d).inverse

        // Connect to division
        (a * c) * (b * d).inverse = a * c * (b * d).inverse

        (a / b) * (c / d) = (a * c) / (b * d)
    }
}

/// Dividing by a fraction flips it: (a/b) / (c/d) = (a/b) * (d/c).
theorem div_by_fraction(a: Real, b: Real, c: Real, d: Real) {
    c != Real.0 and d != Real.0
    implies
    (a / b) / (c / d) = (a / b) * (d / c)
} by {
    if c != Real.0 and d != Real.0 {
        (a / b) / (c / d) = (a / b) * (c / d).inverse
        (c / d).inverse = d / c
        (a / b) / (c / d) = (a / b) * (d / c)
    }
}

/// Division of fractions: (a/b) / (c/d) = (a*d) / (b*c) when b, c, d != 0.
theorem div_div(a: Real, b: Real, c: Real, d: Real) {
    b != Real.0 and c != Real.0 and d != Real.0
    implies
    (a / b) / (c / d) = (a * d) / (b * c)
} by {
    if b != Real.0 and c != Real.0 and d != Real.0 {
        div_by_fraction(a, b, c, d)
        mul_div(a, b, d, c)
        (a / b) / (c / d) = (a / b) * (d / c)
        (a / b) * (d / c) = (a * d) / (b * c)
        (a / b) / (c / d) = (a * d) / (b * c)
    }
}

/// Specific cancellation pattern: [(a*b)/(c*d)] / [b/d] = a/c when b, c, d != 0.
/// This is a direct consequence of div_div and simplification.
/// Currently commented out due to timeout on rearrangement steps.
// theorem div_cancel_pattern(a: Real, b: Real, c: Real, d: Real) {
//     b != Real.0 and c != Real.0 and d != Real.0
//     implies
//     ((a * b) / (c * d)) / (b / d) = a / c
// } by {
//     if b != Real.0 and c != Real.0 and d != Real.0 {
//         // Use div_div: [(a*b)/(c*d)] / [b/d] = [(a*b)*d] / [(c*d)*b]
//         c * d != Real.0
//         ((a * b) / (c * d)) / (b / d) = ((a * b) * d) / ((c * d) * b)
//
//         // Rearrange: (a*b*d) / (c*d*b) = (a*(b*d)) / (c*(d*b))
//         ((a * b) * d) / ((c * d) * b) = (a * (b * d)) / (c * (d * b))
//
//         // Use commutativity: d*b = b*d
//         (a * (b * d)) / (c * (d * b)) = (a * (b * d)) / (c * (b * d))
//
//         // Apply div_cancel_common: (a * b) / (c * b) = a / c
//         b * d != Real.0
//         (a * (b * d)) / (c * (b * d)) = a / c
//     }
// }

theorem geom_series(r: Real) {
    r.abs < 1 implies
    limit(partial(r.pow)) = 1 / (1 - r)
} by {
    1 = (1 - r) * limit(partial(r.pow))
}
