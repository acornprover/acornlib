/// Series convergence consumers for linear combinations of real sequences.
///
/// This module packages shallow algebraic consequences for ordinary series
/// convergence and absolute convergence. It intentionally stays in the series
/// layer: no sequence-side asymptotic algebra, no imports from prior majorant
/// lanes, no asymptotic notation, and no generic eventual API.

from algebra.add_semigroup import add_fn
from list import partial
from nat import Nat
from algebra.semigroup import mul_fn
from real.real_base import Real
from real.real_series import add_seq, add_seq_converges, converges, converges_mul_seq,
    mul_seq, neg_seq, partial_add_seq_comm, partial_mul_seq_comm, tail
from real.abs_conv import absolutely_converges, absolutely_converges_add,
    absolutely_converges_scalar_mul, sub_seq
from real.series_comparison_bridge import absolutely_converges_of_tail,
    series_converges_of_tail

numerals Real

/// Scaling the terms of a convergent series preserves convergence.
theorem series_scalar_mul_converges(c: Real, a: Nat -> Real) {
    converges(partial(a)) implies converges(partial(mul_seq(c, a)))
} by {
    if converges(partial(a)) {
        partial_mul_seq_comm(c, a)
        partial(mul_seq(c, a)) = mul_seq(c, partial(a))
        converges_mul_seq(c, partial(a))
        converges(mul_seq(c, partial(a)))
        converges(partial(mul_seq(c, a)))
    }
}

/// Negating the terms of a convergent series preserves convergence.
theorem series_neg_converges(a: Nat -> Real) {
    converges(partial(a)) implies converges(partial(neg_seq(a)))
} by {
    if converges(partial(a)) {
        series_scalar_mul_converges(-Real.1, a)
        neg_seq(a) = mul_seq(-Real.1, a)
        converges(partial(neg_seq(a)))
    }
}

/// Adding two convergent series termwise preserves convergence.
theorem series_add_converges(a: Nat -> Real, b: Nat -> Real) {
    converges(partial(a)) and converges(partial(b)) implies converges(partial(add_seq(a, b)))
} by {
    if converges(partial(a)) and converges(partial(b)) {
        add_seq_converges(partial(a), partial(b))
        converges(add_seq(partial(a), partial(b)))
        partial_add_seq_comm(a, b)
        partial(add_seq(a, b)) = add_seq(partial(a), partial(b))
        converges(partial(add_seq(a, b)))
    }
}

/// Subtracting two convergent series termwise preserves convergence.
theorem series_sub_converges(a: Nat -> Real, b: Nat -> Real) {
    converges(partial(a)) and converges(partial(b)) implies converges(partial(sub_seq(a, b)))
} by {
    if converges(partial(a)) and converges(partial(b)) {
        forall(k: Nat) {
            sub_seq(a, b, k) = a(k) - b(k)
            neg_seq(b, k) = -b(k)
            add_seq(a, neg_seq(b), k) = a(k) + neg_seq(b, k)
            add_seq(a, neg_seq(b), k) = a(k) + -b(k)
            a(k) + -b(k) = a(k) - b(k)
            sub_seq(a, b, k) = add_seq(a, neg_seq(b), k)
        }
        sub_seq(a, b) = add_seq(a, neg_seq(b))
        series_neg_converges(b)
        converges(partial(neg_seq(b)))
        series_add_converges(a, neg_seq(b))
        converges(partial(add_seq(a, neg_seq(b))))
        converges(partial(sub_seq(a, b)))
    }
}

/// A scalar multiple plus another convergent series is convergent.
theorem series_scalar_add_converges(a: Nat -> Real, b: Nat -> Real, c: Real) {
    converges(partial(a)) and converges(partial(b)) implies
        converges(partial(add_seq(mul_seq(c, a), b)))
} by {
    if converges(partial(a)) and converges(partial(b)) {
        series_scalar_mul_converges(c, a)
        converges(partial(mul_seq(c, a)))
        series_add_converges(mul_seq(c, a), b)
        converges(partial(add_seq(mul_seq(c, a), b)))
    }
}

/// A two-term scalar linear combination of convergent series is convergent.
theorem series_linear_combination_two_converges(a: Nat -> Real, b: Nat -> Real, c: Real, d: Real) {
    converges(partial(a)) and converges(partial(b)) implies
        converges(partial(add_seq(mul_seq(c, a), mul_seq(d, b))))
} by {
    if converges(partial(a)) and converges(partial(b)) {
        series_scalar_mul_converges(c, a)
        converges(partial(mul_seq(c, a)))
        series_scalar_mul_converges(d, b)
        converges(partial(mul_seq(d, b)))
        series_add_converges(mul_seq(c, a), mul_seq(d, b))
        converges(partial(add_seq(mul_seq(c, a), mul_seq(d, b))))
    }
}

/// A difference of scalar multiples of convergent series is convergent.
theorem series_difference_of_scalar_muls_converges(a: Nat -> Real, b: Nat -> Real, c: Real, d: Real) {
    converges(partial(a)) and converges(partial(b)) implies
        converges(partial(sub_seq(mul_seq(c, a), mul_seq(d, b))))
} by {
    if converges(partial(a)) and converges(partial(b)) {
        series_scalar_mul_converges(c, a)
        converges(partial(mul_seq(c, a)))
        series_scalar_mul_converges(d, b)
        converges(partial(mul_seq(d, b)))
        series_sub_converges(mul_seq(c, a), mul_seq(d, b))
        converges(partial(sub_seq(mul_seq(c, a), mul_seq(d, b))))
    }
}

/// Scaling the terms of an absolutely convergent series preserves absolute convergence.
theorem series_scalar_mul_absolutely_converges(c: Real, a: Nat -> Real) {
    absolutely_converges(a) implies absolutely_converges(mul_seq(c, a))
} by {
    if absolutely_converges(a) {
        absolutely_converges_scalar_mul(c, a)
        absolutely_converges(mul_fn(c, a))
        forall(k: Nat) {
            mul_fn(c, a, k) = c * a(k)
            mul_seq(c, a, k) = c * a(k)
            mul_fn(c, a, k) = mul_seq(c, a, k)
        }
        mul_fn(c, a) = mul_seq(c, a)
        absolutely_converges(mul_seq(c, a))
    }
}

/// Negating the terms of an absolutely convergent series preserves absolute convergence.
theorem series_neg_absolutely_converges(a: Nat -> Real) {
    absolutely_converges(a) implies absolutely_converges(neg_seq(a))
} by {
    if absolutely_converges(a) {
        series_scalar_mul_absolutely_converges(-Real.1, a)
        absolutely_converges(mul_seq(-Real.1, a))
        neg_seq(a) = mul_seq(-Real.1, a)
        absolutely_converges(neg_seq(a))
    }
}

/// Adding two absolutely convergent series termwise preserves absolute convergence.
theorem series_add_absolutely_converges(a: Nat -> Real, b: Nat -> Real) {
    absolutely_converges(a) and absolutely_converges(b) implies absolutely_converges(add_seq(a, b))
} by {
    if absolutely_converges(a) and absolutely_converges(b) {
        absolutely_converges_add(a, b)
        absolutely_converges(add_fn(a, b))
        forall(k: Nat) {
            add_fn(a, b, k) = a(k) + b(k)
            add_seq(a, b, k) = a(k) + b(k)
            add_fn(a, b, k) = add_seq(a, b, k)
        }
        add_fn(a, b) = add_seq(a, b)
        absolutely_converges(add_seq(a, b))
    }
}

/// Subtracting two absolutely convergent series termwise preserves absolute convergence.
theorem series_sub_absolutely_converges(a: Nat -> Real, b: Nat -> Real) {
    absolutely_converges(a) and absolutely_converges(b) implies absolutely_converges(sub_seq(a, b))
} by {
    if absolutely_converges(a) and absolutely_converges(b) {
        series_neg_absolutely_converges(b)
        absolutely_converges(neg_seq(b))
        series_add_absolutely_converges(a, neg_seq(b))
        absolutely_converges(add_seq(a, neg_seq(b)))
        forall(k: Nat) {
            sub_seq(a, b, k) = a(k) - b(k)
            neg_seq(b, k) = -b(k)
            add_seq(a, neg_seq(b), k) = a(k) + neg_seq(b, k)
            add_seq(a, neg_seq(b), k) = a(k) + -b(k)
            a(k) + -b(k) = a(k) - b(k)
            sub_seq(a, b, k) = add_seq(a, neg_seq(b), k)
        }
        sub_seq(a, b) = add_seq(a, neg_seq(b))
        absolutely_converges(sub_seq(a, b))
    }
}

/// A scalar multiple plus another absolutely convergent series is absolutely convergent.
theorem series_scalar_add_absolutely_converges(a: Nat -> Real, b: Nat -> Real, c: Real) {
    absolutely_converges(a) and absolutely_converges(b) implies
        absolutely_converges(add_seq(mul_seq(c, a), b))
} by {
    if absolutely_converges(a) and absolutely_converges(b) {
        series_scalar_mul_absolutely_converges(c, a)
        absolutely_converges(mul_seq(c, a))
        series_add_absolutely_converges(mul_seq(c, a), b)
        absolutely_converges(add_seq(mul_seq(c, a), b))
    }
}

/// A two-term scalar linear combination of absolutely convergent series is absolutely convergent.
theorem series_linear_combination_two_absolutely_converges(a: Nat -> Real, b: Nat -> Real, c: Real, d: Real) {
    absolutely_converges(a) and absolutely_converges(b) implies
        absolutely_converges(add_seq(mul_seq(c, a), mul_seq(d, b)))
} by {
    if absolutely_converges(a) and absolutely_converges(b) {
        series_scalar_mul_absolutely_converges(c, a)
        absolutely_converges(mul_seq(c, a))
        series_scalar_mul_absolutely_converges(d, b)
        absolutely_converges(mul_seq(d, b))
        series_add_absolutely_converges(mul_seq(c, a), mul_seq(d, b))
        absolutely_converges(add_seq(mul_seq(c, a), mul_seq(d, b)))
    }
}

/// A difference of scalar multiples of absolutely convergent series is absolutely convergent.
theorem series_difference_of_scalar_muls_absolutely_converges(
    a: Nat -> Real,
    b: Nat -> Real,
    c: Real,
    d: Real
) {
    absolutely_converges(a) and absolutely_converges(b) implies
        absolutely_converges(sub_seq(mul_seq(c, a), mul_seq(d, b)))
} by {
    if absolutely_converges(a) and absolutely_converges(b) {
        series_scalar_mul_absolutely_converges(c, a)
        absolutely_converges(mul_seq(c, a))
        series_scalar_mul_absolutely_converges(d, b)
        absolutely_converges(mul_seq(d, b))
        series_sub_absolutely_converges(mul_seq(c, a), mul_seq(d, b))
        absolutely_converges(sub_seq(mul_seq(c, a), mul_seq(d, b)))
    }
}

/// Tail convergence of two series implies convergence of their termwise difference.
theorem series_sub_converges_of_tails(a: Nat -> Real, b: Nat -> Real, n: Nat) {
    converges(partial(tail(a, n))) and converges(partial(tail(b, n))) implies
        converges(partial(sub_seq(a, b)))
} by {
    if converges(partial(tail(a, n))) and converges(partial(tail(b, n))) {
        series_converges_of_tail(a, n)
        converges(partial(a))
        series_converges_of_tail(b, n)
        converges(partial(b))
        series_sub_converges(a, b)
        converges(partial(sub_seq(a, b)))
    }
}

/// Tail convergence of two series implies convergence of their two-term linear combination.
theorem series_linear_combination_two_converges_of_tails(
    a: Nat -> Real,
    b: Nat -> Real,
    c: Real,
    d: Real,
    n: Nat
) {
    converges(partial(tail(a, n))) and converges(partial(tail(b, n))) implies
        converges(partial(add_seq(mul_seq(c, a), mul_seq(d, b))))
} by {
    if converges(partial(tail(a, n))) and converges(partial(tail(b, n))) {
        series_converges_of_tail(a, n)
        converges(partial(a))
        series_converges_of_tail(b, n)
        converges(partial(b))
        series_linear_combination_two_converges(a, b, c, d)
        converges(partial(add_seq(mul_seq(c, a), mul_seq(d, b))))
    }
}

/// Absolute convergence of two tails implies absolute convergence of the termwise difference.
theorem series_sub_absolutely_converges_of_tails(a: Nat -> Real, b: Nat -> Real, n: Nat) {
    absolutely_converges(tail(a, n)) and absolutely_converges(tail(b, n)) implies
        absolutely_converges(sub_seq(a, b))
} by {
    if absolutely_converges(tail(a, n)) and absolutely_converges(tail(b, n)) {
        absolutely_converges_of_tail(a, n)
        absolutely_converges(a)
        absolutely_converges_of_tail(b, n)
        absolutely_converges(b)
        series_sub_absolutely_converges(a, b)
        absolutely_converges(sub_seq(a, b))
    }
}

/// Absolute convergence of two tails implies absolute convergence of their two-term linear combination.
theorem series_linear_combination_two_absolutely_converges_of_tails(
    a: Nat -> Real,
    b: Nat -> Real,
    c: Real,
    d: Real,
    n: Nat
) {
    absolutely_converges(tail(a, n)) and absolutely_converges(tail(b, n)) implies
        absolutely_converges(add_seq(mul_seq(c, a), mul_seq(d, b)))
} by {
    if absolutely_converges(tail(a, n)) and absolutely_converges(tail(b, n)) {
        absolutely_converges_of_tail(a, n)
        absolutely_converges(a)
        absolutely_converges_of_tail(b, n)
        absolutely_converges(b)
        series_linear_combination_two_absolutely_converges(a, b, c, d)
        absolutely_converges(add_seq(mul_seq(c, a), mul_seq(d, b)))
    }
}
