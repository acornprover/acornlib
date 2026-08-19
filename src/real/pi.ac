from nat import Nat, lte_mul_both, mul_comm, from_nat, lt_add_suc, lt_suc, pow_add, lte_add_left
from rat import Rat, nat_lt_imp_rat_lt
from order import lte_refl, lte_trans, lt_trans, lt_imp_lte, not_lt_imp_gte, not_lte_imp_gt, not_gte_imp_lt, lte_antisymm, lt_of_lte_of_lt, lt_of_lt_of_lte
from list import partial, partial_split_last
from data.basic.set import Set, set_ext
from real.real_field import Real, mul_div, real_no_zero_divisors, zero_is_different_than_one
from real.real_ring import converges, limit, converges_to, mul_abs, mul_zero_left, mul_zero_right, real_mul_comm, lte_mul_nonneg_right, exists_small_mul_variant, lt_mul_pos_left, mul_nonneg, mul_pos_pos, mul_from_rat, lt_mul_pos_right, mul_distrib_left, mul_distrib_right, mul_sub_distrib_left
from real.real_seq import eq_imp_limit, eventual_eq, converges_to_imp_converges, converges_imp_converges_to, converges_to_unique, add_seq, eps_smaller_than_both, close_and_lt_imp_close
from real.real_base import abs_neg, neg_neg, add_comm, add_assoc, abs_gte_zero, lte_abs, pos_imp_eq_abs, lte_lt_trans, lt_add_right, lte_add_right, bounds_imp_close, self_close, from_rat_maintains_lt, lt_add_pos
from real.abs_conv import absolutely_converges, abs_fn, absolutely_converges_imp_converges, sub_seq
from real.real_series import partial_suc, partial_zero, partial_one, tail, tail_imp_converges_to, const_converges, const_limit, const_seq, mul_seq, neg_seq, seq_lte, seq_lte_preserves_limit, partial_seq_lte, partial_tail_sub, partial_tail_decomp, triangle_ineq, partial_tail_conv_imp_partial_conv, tail_partial_converges, is_lower_bound, is_upper_bound, nonneg_partial_increasing, is_increasing, increasing_convergent_bounded_by_limit, increasing_bounded_above_converges, pow_nonneg, abs_pow, has_upper_bound_seq
from real.limits import is_subsequence_index, subsequence, converges_subsequence, converges_subsequence_add, is_monotone, is_unbounded, subsequence_index_of_monotone_unbounded
from real.cauchy_criterion import is_cauchy_seq, converges_to_imp_cauchy
from real.exp import exp_term, factorial_pos, pow_suc, zero_pow_pos, inverse_pos, abs_div, two, mul_frac_assoc, mul_assoc_real, two_positive, factorial_suc_real, exp_term_mul_recurrence, exp_term_pos, div_lt_div_pos, suc_pos, rat_from_nat_mul
from real.pi_helpers import cos_two_tail_pair_index_odd, cos_two_tail_pair_index_even, cos_two_tail_double_suc_index, cos_two_tail_odd_suc, cos_two_tail_odd_add_two, cos_two_tail_odd_ge_one, cos_two_tail_odd_partial_index, cos_two_tail_index_add_two, cos_two_tail_index_add_one, nat_mul_2_8, nat_mul_3_8
from real.trig import sin_term, cos_term, sin_term_abs_converges, cos_term_abs_converges, cos_zero, sin_zero, two_mul_suc, alternating_sign_abs, real_pow_mul_distrib, alternating_sign_double, alternating_sign_double_suc, cos_term_abs, cos_term_abs_exp
from real.trig_identities import sin_add, cos_add, sin_sq_add_cos_sq, sin_neg, cos_neg
from real.derivative_trig import cos_sub_one_series_small, sin_series_small, cos_shift_sum_bound, sin_shift_sum_close_one, half_lt_one, half_pos, one_div_two_eq_half, one_div_one_half_eq_two, sin_is_derivative_fn, abs_of_nonneg, sq_lte_base, sin_shift_sum, cos_shift_sum
from real.derivative_basic import has_derivative_at, differentiable_at, difference_quotient, has_derivative_at_unique
from real.derivative_continuity import derivative_continuous_at
from real.calculus_api import is_derivative_fn, is_derivative_fn_at
from real.calculus_quotient_continuity_examples import is_derivative_fn_imp_continuous
from real.mean_value import mean_value_theorem, secant_slope
from real.intermediate_value import intermediate_value_closed_interval
from real.supremum import is_nonempty, is_set_upper_bound, has_upper_bound, is_set_lower_bound, is_set_supremum, is_set_infimum, completeness, set_supremum_close_from_below, set_member_le_supremum, set_supremum_le_upper_bound, set_upper_bound_contains_le, set_supremum_is_upper_bound, supremum_infimum_duality, negate_set, negate_set_contains, set_bound_below_supremum_not_upper, set_not_upper_bound_witness
from real.continuity_base import continuous, continuous_at, continuous_condition
from order_set import closed_interval_set, closed_interval_set_contains_eq
from order import closed_interval
from algebra.ring.ring import alternating_sign, alternating_sign_zero, alternating_sign_suc

numerals Real
numerals Nat

// Section 1: Continuity of the trigonometric functions.

/// The sine function is continuous everywhere.
theorem sin_continuous {
    continuous(Real.sin)
} by {
    sin_is_derivative_fn
    is_derivative_fn_imp_continuous(Real.sin, Real.cos)
    continuous(Real.sin)
}

/// The absolute value of the difference of cosine at a small argument from one.
theorem cos_shift_diff_bound(h: Real) {
    h.abs < Real.1 / two implies (h.cos - Real.1).abs <= two * h.abs.pow(Nat.2)
} by {
    if h.abs < Real.1 / two {
        lt_imp_lte(h.abs, Real.1 / two)
        h.abs <= Real.1 / two
        half_lt_one
        Real.1 / two < Real.1
        lt_of_lte_of_lt(h.abs, Real.1 / two, Real.1)
        h.abs < Real.1
        cos_sub_one_series_small(h)
        h.cos - Real.1 = h.pow(Nat.2) * cos_shift_sum(h)
        (h.cos - Real.1).abs = (h.pow(Nat.2) * cos_shift_sum(h)).abs
        mul_abs(h.pow(Nat.2), cos_shift_sum(h))
        (h.pow(Nat.2) * cos_shift_sum(h)).abs = h.pow(Nat.2).abs * cos_shift_sum(h).abs
        abs_pow(h, Nat.2)
        h.pow(Nat.2).abs = h.abs.pow(Nat.2)
        (h.pow(Nat.2) * cos_shift_sum(h)).abs = h.abs.pow(Nat.2) * cos_shift_sum(h).abs
        cos_shift_sum_bound(h)
        cos_shift_sum(h).abs <= two
        abs_gte_zero(h.abs.pow(Nat.2))
        lte_mul_nonneg_right(cos_shift_sum(h).abs, two, h.abs.pow(Nat.2))
        cos_shift_sum(h).abs * h.abs.pow(Nat.2) <= two * h.abs.pow(Nat.2)
        h.abs.pow(Nat.2) * cos_shift_sum(h).abs <= two * h.abs.pow(Nat.2)
        (h.cos - Real.1).abs <= two * h.abs.pow(Nat.2)
    }
}

/// The absolute value of sine at a small argument.
theorem sin_small_bound(h: Real) {
    h.abs < Real.1 / two implies h.sin.abs <= h.abs * (two * h.abs.pow(Nat.2) + Real.1)
} by {
    if h.abs < Real.1 / two {
        lt_imp_lte(h.abs, Real.1 / two)
        h.abs <= Real.1 / two
        half_lt_one
        Real.1 / two < Real.1
        lt_of_lte_of_lt(h.abs, Real.1 / two, Real.1)
        h.abs < Real.1
        sin_series_small(h)
        h.sin = h * sin_shift_sum(h)
        h.sin.abs = (h * sin_shift_sum(h)).abs
        mul_abs(h, sin_shift_sum(h))
        (h * sin_shift_sum(h)).abs = h.abs * sin_shift_sum(h).abs
        h.sin.abs = h.abs * sin_shift_sum(h).abs
        sin_shift_sum_close_one(h)
        (sin_shift_sum(h) - Real.1).abs <= two * h.abs.pow(Nat.2)
        triangle_ineq(sin_shift_sum(h) - Real.1, Real.1)
        ((sin_shift_sum(h) - Real.1) + Real.1).abs <= (sin_shift_sum(h) - Real.1).abs + Real.1.abs
        Real.1.abs = Real.1
        (sin_shift_sum(h) - Real.1) + Real.1 = sin_shift_sum(h)
        sin_shift_sum(h).abs <= (sin_shift_sum(h) - Real.1).abs + Real.1
        lte_add_right((sin_shift_sum(h) - Real.1).abs, two * h.abs.pow(Nat.2), Real.1)
        (sin_shift_sum(h) - Real.1).abs + Real.1 <= two * h.abs.pow(Nat.2) + Real.1
        lte_trans(sin_shift_sum(h).abs, (sin_shift_sum(h) - Real.1).abs + Real.1, two * h.abs.pow(Nat.2) + Real.1)
        sin_shift_sum(h).abs <= two * h.abs.pow(Nat.2) + Real.1
        abs_gte_zero(h)
        lte_mul_nonneg_right(sin_shift_sum(h).abs, two * h.abs.pow(Nat.2) + Real.1, h.abs)
        sin_shift_sum(h).abs * h.abs <= (two * h.abs.pow(Nat.2) + Real.1) * h.abs
        h.abs * sin_shift_sum(h).abs <= h.abs * (two * h.abs.pow(Nat.2) + Real.1)
        h.sin.abs <= h.abs * (two * h.abs.pow(Nat.2) + Real.1)
    }
}

/// The cosine of a sum differs from the cosine of the base point by a signed combination.
theorem cos_sub_identity(x0: Real, h: Real) {
    (x0 + h).cos - x0.cos = x0.cos * (h.cos - Real.1) - x0.sin * h.sin
} by {
    cos_add(x0, h)
    (x0 + h).cos = x0.cos * h.cos - x0.sin * h.sin
    (x0 + h).cos - x0.cos = x0.cos * h.cos - x0.sin * h.sin - x0.cos
    x0.cos * h.cos - x0.sin * h.sin - x0.cos =
        x0.cos * h.cos - x0.cos - x0.sin * h.sin
    x0.cos * h.cos - x0.cos = x0.cos * (h.cos - Real.1)
    x0.cos * h.cos - x0.cos - x0.sin * h.sin =
        x0.cos * (h.cos - Real.1) - x0.sin * h.sin
    (x0 + h).cos - x0.cos = x0.cos * (h.cos - Real.1) - x0.sin * h.sin
}

/// The absolute difference of cosine is bounded by a linear multiple of the argument difference.
/// The absolute difference of cosine is bounded by the triangle inequality applied to the
/// addition formula.
theorem cos_abs_triangle(x0: Real, h: Real) {
    ((x0 + h).cos - x0.cos).abs <= (x0.cos * (h.cos - Real.1)).abs + (x0.sin * h.sin).abs
} by {
    cos_sub_identity(x0, h)
    (x0 + h).cos - x0.cos = x0.cos * (h.cos - Real.1) - x0.sin * h.sin
    triangle_ineq(x0.cos * (h.cos - Real.1), -(x0.sin * h.sin))
}

/// The absolute difference of cosine is bounded by a linear multiple of the argument difference.
theorem cos_abs_diff_bound(x0: Real, x: Real) {
    (x - x0).abs < Real.1 / two implies
    (x.cos - x0.cos).abs <= (two * (x0.sin.abs + x0.cos.abs)) * (x - x0).abs
} by {
    if (x - x0).abs < Real.1 / two {
        x - x0 = x + -x0
        x0 + (x - x0) = x0 + (x + -x0)
        add_comm(x0, x + -x0)
        x0 + (x + -x0) = (x + -x0) + x0
        (x + -x0) + x0 = x + (-x0 + x0)
        -x0 + x0 = Real.0
        x + (-x0 + x0) = x + Real.0
        x + Real.0 = x
        x0 + (x - x0) = x
        cos_abs_triangle(x0, x - x0)
        ((x0 + (x - x0)).cos - x0.cos).abs <= (x0.cos * ((x - x0).cos - Real.1)).abs + (x0.sin * (x - x0).sin).abs
        (x0 + (x - x0)).cos = x.cos
        ((x0 + (x - x0)).cos - x0.cos).abs = (x.cos - x0.cos).abs
        (x.cos - x0.cos).abs <= (x0.cos * ((x - x0).cos - Real.1)).abs + (x0.sin * (x - x0).sin).abs
        mul_abs(x0.cos, (x - x0).cos - Real.1)
        (x0.cos * ((x - x0).cos - Real.1)).abs = x0.cos.abs * ((x - x0).cos - Real.1).abs
        mul_abs(x0.sin, (x - x0).sin)
        (x0.sin * (x - x0).sin).abs = x0.sin.abs * (x - x0).sin.abs
        (x.cos - x0.cos).abs <= x0.cos.abs * ((x - x0).cos - Real.1).abs + x0.sin.abs * (x - x0).sin.abs
        cos_shift_diff_bound(x - x0)
        ((x - x0).cos - Real.1).abs <= two * (x - x0).abs.pow(Nat.2)
        abs_gte_zero(x0.cos)
        lte_mul_nonneg_right(((x - x0).cos - Real.1).abs, two * (x - x0).abs.pow(Nat.2), x0.cos.abs)
        x0.cos.abs * ((x - x0).cos - Real.1).abs <= x0.cos.abs * (two * (x - x0).abs.pow(Nat.2))
        x0.cos.abs * (two * (x - x0).abs.pow(Nat.2)) = two * x0.cos.abs * (x - x0).abs.pow(Nat.2)
        x0.cos.abs * ((x - x0).cos - Real.1).abs <= two * x0.cos.abs * (x - x0).abs.pow(Nat.2)
        sin_small_bound(x - x0)
        (x - x0).sin.abs <= (x - x0).abs * (two * (x - x0).abs.pow(Nat.2) + Real.1)
        abs_gte_zero(x0.sin)
        lte_mul_nonneg_right((x - x0).sin.abs, (x - x0).abs * (two * (x - x0).abs.pow(Nat.2) + Real.1), x0.sin.abs)
        x0.sin.abs * (x - x0).sin.abs <= x0.sin.abs * ((x - x0).abs * (two * (x - x0).abs.pow(Nat.2) + Real.1))
        x0.sin.abs * ((x - x0).abs * (two * (x - x0).abs.pow(Nat.2) + Real.1)) =
            x0.sin.abs * (x - x0).abs * (two * (x - x0).abs.pow(Nat.2) + Real.1)
        x0.sin.abs * (x - x0).sin.abs <= x0.sin.abs * (x - x0).abs * (two * (x - x0).abs.pow(Nat.2) + Real.1)
        lte_add_right(x0.cos.abs * ((x - x0).cos - Real.1).abs,
            two * x0.cos.abs * (x - x0).abs.pow(Nat.2),
            x0.sin.abs * (x - x0).sin.abs)
        x0.cos.abs * ((x - x0).cos - Real.1).abs + x0.sin.abs * (x - x0).sin.abs <= two * x0.cos.abs * (x - x0).abs.pow(Nat.2) + x0.sin.abs * (x - x0).sin.abs
        lte_add_right(x0.sin.abs * (x - x0).sin.abs,
            x0.sin.abs * (x - x0).abs * (two * (x - x0).abs.pow(Nat.2) + Real.1),
            two * x0.cos.abs * (x - x0).abs.pow(Nat.2))
        two * x0.cos.abs * (x - x0).abs.pow(Nat.2) + x0.sin.abs * (x - x0).sin.abs <= two * x0.cos.abs * (x - x0).abs.pow(Nat.2) +
            x0.sin.abs * (x - x0).abs * (two * (x - x0).abs.pow(Nat.2) + Real.1)
        lte_trans(x0.cos.abs * ((x - x0).cos - Real.1).abs + x0.sin.abs * (x - x0).sin.abs,
            two * x0.cos.abs * (x - x0).abs.pow(Nat.2) + x0.sin.abs * (x - x0).sin.abs,
            two * x0.cos.abs * (x - x0).abs.pow(Nat.2) +
            x0.sin.abs * (x - x0).abs * (two * (x - x0).abs.pow(Nat.2) + Real.1))
        x0.cos.abs * ((x - x0).cos - Real.1).abs + x0.sin.abs * (x - x0).sin.abs <= two * x0.cos.abs * (x - x0).abs.pow(Nat.2) + x0.sin.abs * (x - x0).abs * (two * (x - x0).abs.pow(Nat.2) + Real.1)
        lte_trans((x.cos - x0.cos).abs,
            x0.cos.abs * ((x - x0).cos - Real.1).abs + x0.sin.abs * (x - x0).sin.abs,
            two * x0.cos.abs * (x - x0).abs.pow(Nat.2) +
            x0.sin.abs * (x - x0).abs * (two * (x - x0).abs.pow(Nat.2) + Real.1))
        (x.cos - x0.cos).abs <= two * x0.cos.abs * (x - x0).abs.pow(Nat.2) + x0.sin.abs * (x - x0).abs * (two * (x - x0).abs.pow(Nat.2) + Real.1)
        abs_gte_zero(x - x0)
        (x - x0).abs >= Real.0
        lt_imp_lte((x - x0).abs, Real.1 / two)
        (x - x0).abs <= Real.1 / two
        half_lt_one
        Real.1 / two < Real.1
        lt_of_lte_of_lt((x - x0).abs, Real.1 / two, Real.1)
        (x - x0).abs < Real.1
        lt_imp_lte((x - x0).abs, Real.1)
        (x - x0).abs <= Real.1
        sq_lte_base((x - x0).abs)
        (x - x0).abs.pow(Nat.2) <= (x - x0).abs
        lte_mul_nonneg_right((x - x0).abs.pow(Nat.2), (x - x0).abs, two)
        (x - x0).abs.pow(Nat.2) * two <= (x - x0).abs * two
        two * (x - x0).abs.pow(Nat.2) <= two * (x - x0).abs
        lte_mul_nonneg_right(two * (x - x0).abs.pow(Nat.2), two * (x - x0).abs, x0.cos.abs)
        x0.cos.abs * (two * (x - x0).abs.pow(Nat.2)) <= x0.cos.abs * (two * (x - x0).abs)
        two * x0.cos.abs * (x - x0).abs.pow(Nat.2) <= two * x0.cos.abs * (x - x0).abs
        lte_add_right(two * (x - x0).abs.pow(Nat.2), two * (x - x0).abs, Real.1)
        two * (x - x0).abs.pow(Nat.2) + Real.1 <= two * (x - x0).abs + Real.1
        lt_imp_lte((x - x0).abs, Real.1 / two)
        (x - x0).abs <= Real.1 / two
        lte_mul_nonneg_right((x - x0).abs, Real.1 / two, two)
        (x - x0).abs * two <= (Real.1 / two) * two
        two * (x - x0).abs <= two * (Real.1 / two)
        Real.1 / two * two = Real.1
        two * (Real.1 / two) = Real.1
        two * (x - x0).abs <= Real.1
        lte_add_right(two * (x - x0).abs, Real.1, Real.1)
        two * (x - x0).abs + Real.1 <= Real.1 + Real.1
        Real.1 + Real.1 = two
        two * (x - x0).abs + Real.1 <= two
        lte_trans(two * (x - x0).abs.pow(Nat.2) + Real.1, two * (x - x0).abs + Real.1, two)
        two * (x - x0).abs.pow(Nat.2) + Real.1 <= two
        lte_mul_nonneg_right(two * (x - x0).abs.pow(Nat.2) + Real.1, two, (x - x0).abs)
        (two * (x - x0).abs.pow(Nat.2) + Real.1) * (x - x0).abs <= two * (x - x0).abs
        abs_gte_zero(x0.sin)
        x0.sin.abs >= Real.0
        abs_gte_zero(x - x0)
        (x - x0).abs >= Real.0
        mul_nonneg(x0.sin.abs, (x - x0).abs)
        x0.sin.abs * (x - x0).abs >= Real.0
        not (x0.sin.abs * (x - x0).abs).is_negative
        lte_mul_nonneg_right(two * (x - x0).abs.pow(Nat.2) + Real.1, two, x0.sin.abs * (x - x0).abs)
        (two * (x - x0).abs.pow(Nat.2) + Real.1) * (x0.sin.abs * (x - x0).abs) <= two * (x0.sin.abs * (x - x0).abs)
        (two * (x - x0).abs.pow(Nat.2) + Real.1) * (x0.sin.abs * (x - x0).abs) = x0.sin.abs * (x - x0).abs * (two * (x - x0).abs.pow(Nat.2) + Real.1)
        two * (x0.sin.abs * (x - x0).abs) = two * x0.sin.abs * (x - x0).abs
        x0.sin.abs * (x - x0).abs * (two * (x - x0).abs.pow(Nat.2) + Real.1) <= two * x0.sin.abs * (x - x0).abs
        x0.sin.abs * (x - x0).abs * (two * (x - x0).abs.pow(Nat.2) + Real.1) <= x0.sin.abs * (x - x0).abs * two
        x0.sin.abs * (x - x0).abs * two = two * x0.sin.abs * (x - x0).abs
        x0.sin.abs * (x - x0).abs * (two * (x - x0).abs.pow(Nat.2) + Real.1) <= two * x0.sin.abs * (x - x0).abs
        lte_add_right(two * x0.cos.abs * (x - x0).abs.pow(Nat.2),
            two * x0.cos.abs * (x - x0).abs,
            x0.sin.abs * (x - x0).abs * (two * (x - x0).abs.pow(Nat.2) + Real.1))
        two * x0.cos.abs * (x - x0).abs.pow(Nat.2) +
        x0.sin.abs * (x - x0).abs * (two * (x - x0).abs.pow(Nat.2) + Real.1) <= two * x0.cos.abs * (x - x0).abs +
            x0.sin.abs * (x - x0).abs * (two * (x - x0).abs.pow(Nat.2) + Real.1)
        lte_add_right(x0.sin.abs * (x - x0).abs * (two * (x - x0).abs.pow(Nat.2) + Real.1),
            two * x0.sin.abs * (x - x0).abs, two * x0.cos.abs * (x - x0).abs)
        two * x0.cos.abs * (x - x0).abs +
        x0.sin.abs * (x - x0).abs * (two * (x - x0).abs.pow(Nat.2) + Real.1) <= two * x0.cos.abs * (x - x0).abs + two * x0.sin.abs * (x - x0).abs
        lte_trans(
            two * x0.cos.abs * (x - x0).abs.pow(Nat.2) +
            x0.sin.abs * (x - x0).abs * (two * (x - x0).abs.pow(Nat.2) + Real.1),
            two * x0.cos.abs * (x - x0).abs +
            x0.sin.abs * (x - x0).abs * (two * (x - x0).abs.pow(Nat.2) + Real.1),
            two * x0.cos.abs * (x - x0).abs + two * x0.sin.abs * (x - x0).abs)
        two * x0.cos.abs * (x - x0).abs.pow(Nat.2) +
        x0.sin.abs * (x - x0).abs * (two * (x - x0).abs.pow(Nat.2) + Real.1) <= two * x0.cos.abs * (x - x0).abs + two * x0.sin.abs * (x - x0).abs
        lte_trans((x.cos - x0.cos).abs,
            two * x0.cos.abs * (x - x0).abs.pow(Nat.2) +
            x0.sin.abs * (x - x0).abs * (two * (x - x0).abs.pow(Nat.2) + Real.1),
            two * x0.cos.abs * (x - x0).abs + two * x0.sin.abs * (x - x0).abs)
        (x.cos - x0.cos).abs <= two * x0.cos.abs * (x - x0).abs + two * x0.sin.abs * (x - x0).abs
        two * x0.cos.abs * (x - x0).abs + two * x0.sin.abs * (x - x0).abs =
            (two * x0.cos.abs + two * x0.sin.abs) * (x - x0).abs
        two * x0.cos.abs + two * x0.sin.abs = two * (x0.cos.abs + x0.sin.abs)
        (two * x0.cos.abs + two * x0.sin.abs) * (x - x0).abs =
            (two * (x0.cos.abs + x0.sin.abs)) * (x - x0).abs
        two * (x0.cos.abs + x0.sin.abs) = two * (x0.sin.abs + x0.cos.abs)
        (two * (x0.cos.abs + x0.sin.abs)) * (x - x0).abs =
            (two * (x0.sin.abs + x0.cos.abs)) * (x - x0).abs
        (x.cos - x0.cos).abs <= (two * (x0.sin.abs + x0.cos.abs)) * (x - x0).abs
    }
}

/// The cosine function is continuous at every point.
theorem cos_continuous_at(x0: Real) {
    continuous_at(Real.cos, x0)
} by {
    forall(eps: Real) {
        if eps.is_positive {
            exists_small_mul_variant(two * (x0.sin.abs + x0.cos.abs), eps)
            let delta1: Real satisfy {
                delta1.is_positive and (two * (x0.sin.abs + x0.cos.abs)).abs * delta1 < eps
            }
            half_pos
            Real.1 / two > Real.0
            eps_smaller_than_both(Real.1 / two, delta1)
            let delta: Real satisfy {
                delta.is_positive and delta < Real.1 / two and delta < delta1
            }
            forall(x: Real) {
                if x.is_close(x0, delta) {
                    x.is_close(x0, delta) = (x - x0).abs < delta
                    (x - x0).abs < delta
                    lt_trans((x - x0).abs, delta, Real.1 / two)
                    (x - x0).abs < Real.1 / two
                    lt_trans((x - x0).abs, delta, delta1)
                    (x - x0).abs < delta1
                    cos_abs_diff_bound(x0, x)
                    (x.cos - x0.cos).abs <= (two * (x0.sin.abs + x0.cos.abs)) * (x - x0).abs
                    lte_abs(two * (x0.sin.abs + x0.cos.abs))
                    two * (x0.sin.abs + x0.cos.abs) <= (two * (x0.sin.abs + x0.cos.abs)).abs
                    abs_gte_zero(x - x0)
                    (x - x0).abs >= Real.0
                    lte_mul_nonneg_right(two * (x0.sin.abs + x0.cos.abs),
                        (two * (x0.sin.abs + x0.cos.abs)).abs, (x - x0).abs)
                    (two * (x0.sin.abs + x0.cos.abs)) * (x - x0).abs <= (two * (x0.sin.abs + x0.cos.abs)).abs * (x - x0).abs
                    lte_trans((x.cos - x0.cos).abs,
                        (two * (x0.sin.abs + x0.cos.abs)) * (x - x0).abs,
                        (two * (x0.sin.abs + x0.cos.abs)).abs * (x - x0).abs)
                    (x.cos - x0.cos).abs <= (two * (x0.sin.abs + x0.cos.abs)).abs * (x - x0).abs
                    lt_imp_lte((x - x0).abs, delta1)
                    (x - x0).abs <= delta1
                    abs_gte_zero(two * (x0.sin.abs + x0.cos.abs))
                    lte_mul_nonneg_right((x - x0).abs, delta1, (two * (x0.sin.abs + x0.cos.abs)).abs)
                    (x - x0).abs * (two * (x0.sin.abs + x0.cos.abs)).abs <= delta1 * (two * (x0.sin.abs + x0.cos.abs)).abs
                    (two * (x0.sin.abs + x0.cos.abs)).abs * (x - x0).abs <= (two * (x0.sin.abs + x0.cos.abs)).abs * delta1
                    lte_trans((x.cos - x0.cos).abs,
                        (two * (x0.sin.abs + x0.cos.abs)).abs * (x - x0).abs,
                        (two * (x0.sin.abs + x0.cos.abs)).abs * delta1)
                    (x.cos - x0.cos).abs <= (two * (x0.sin.abs + x0.cos.abs)).abs * delta1
                    lt_imp_lte(delta1, delta1)
                    lte_lt_trans((x.cos - x0.cos).abs,
                        (two * (x0.sin.abs + x0.cos.abs)).abs * delta1, eps)
                    (x.cos - x0.cos).abs < eps
                    x.cos.is_close(x0.cos, eps)
                }
            }
            continuous_condition(Real.cos, x0, delta, eps)
            delta.is_positive and continuous_condition(Real.cos, x0, delta, eps)
            exists(delta2: Real) {
                delta2.is_positive and continuous_condition(Real.cos, x0, delta2, eps)
            }
        }
    }
    continuous_at(Real.cos, x0) = forall(eps: Real) {
        eps.is_positive implies exists(delta: Real) {
            delta.is_positive and continuous_condition(Real.cos, x0, delta, eps)
        }
    }
    if not continuous_at(Real.cos, x0) {
        not forall(eps: Real) {
            eps.is_positive implies exists(delta: Real) {
                delta.is_positive and continuous_condition(Real.cos, x0, delta, eps)
            }
        }
        let bad_eps: Real satisfy {
            bad_eps.is_positive and forall(delta: Real) {
                not (delta.is_positive and continuous_condition(Real.cos, x0, delta, bad_eps))
            }
        }
        exists(delta: Real) {
            delta.is_positive and continuous_condition(Real.cos, x0, delta, bad_eps)
        }
        false
    }
}

/// The cosine function is continuous everywhere.
theorem cos_continuous {
    continuous(Real.cos)
} by {
    forall(x: Real) {
        cos_continuous_at(x)
        continuous_at(Real.cos, x)
    }
    continuous(Real.cos) = forall(x: Real) {
        continuous_at(Real.cos, x)
    }
    continuous(Real.cos)
}

/// The pointwise negation of cosine.
define cos_neg_fn(x: Real) -> Real {
    -x.cos
}

/// The negation of cosine is continuous at every point.
theorem cos_neg_fn_continuous_at(x0: Real) {
    continuous_at(cos_neg_fn, x0)
} by {
    forall(eps: Real) {
        if eps.is_positive {
            cos_continuous_at(x0)
            continuous_at(Real.cos, x0) = forall(eps0: Real) {
                eps0.is_positive implies exists(delta: Real) {
                    delta.is_positive and continuous_condition(Real.cos, x0, delta, eps0)
                }
            }
            let delta: Real satisfy {
                delta.is_positive and continuous_condition(Real.cos, x0, delta, eps)
            }
            forall(x: Real) {
                if x.is_close(x0, delta) {
                    continuous_condition(Real.cos, x0, delta, eps) = forall(x1: Real) {
                        x1.is_close(x0, delta) implies x1.cos.is_close(x0.cos, eps)
                    }
                    x.cos.is_close(x0.cos, eps)
                    (x.cos - x0.cos).abs < eps
                    cos_neg_fn(x) = -x.cos
                    cos_neg_fn(x0) = -x0.cos
                    cos_neg_fn(x) - cos_neg_fn(x0) = -x.cos - (-x0.cos)
                    -x.cos - (-x0.cos) = -(x.cos - x0.cos)
                    abs_neg(x.cos - x0.cos)
                    (-(x.cos - x0.cos)).abs = (x.cos - x0.cos).abs
                    (cos_neg_fn(x) - cos_neg_fn(x0)).abs < eps
                    cos_neg_fn(x).is_close(cos_neg_fn(x0), eps)
                }
            }
            continuous_condition(cos_neg_fn, x0, delta, eps)
            delta.is_positive and continuous_condition(cos_neg_fn, x0, delta, eps)
            exists(delta2: Real) {
                delta2.is_positive and continuous_condition(cos_neg_fn, x0, delta2, eps)
            }
        }
    }
    continuous_at(cos_neg_fn, x0) = forall(eps: Real) {
        eps.is_positive implies exists(delta: Real) {
            delta.is_positive and continuous_condition(cos_neg_fn, x0, delta, eps)
        }
    }
    if not continuous_at(cos_neg_fn, x0) {
        not forall(eps: Real) {
            eps.is_positive implies exists(delta: Real) {
                delta.is_positive and continuous_condition(cos_neg_fn, x0, delta, eps)
            }
        }
        let bad_eps: Real satisfy {
            bad_eps.is_positive and forall(delta: Real) {
                not (delta.is_positive and continuous_condition(cos_neg_fn, x0, delta, bad_eps))
            }
        }
        exists(delta: Real) {
            delta.is_positive and continuous_condition(cos_neg_fn, x0, delta, bad_eps)
        }
        false
    }
}

/// The negation of cosine is continuous everywhere.
theorem cos_neg_fn_continuous {
    continuous(cos_neg_fn)
} by {
    forall(x: Real) {
        cos_neg_fn_continuous_at(x)
        continuous_at(cos_neg_fn, x)
    }
    continuous(cos_neg_fn) = forall(x: Real) {
        continuous_at(cos_neg_fn, x)
    }
    continuous(cos_neg_fn)
}


// Section 2: Cosine is negative at two.

theorem four_lt_twelve {
    Nat.4 < Nat.12
} by {
    lt_suc(Nat.4)
    Nat.4 < Nat.5
    lt_suc(Nat.5)
    Nat.5 < Nat.6
    lt_trans(Nat.4, Nat.5, Nat.6)
    Nat.4 < Nat.6
    lt_suc(Nat.6)
    Nat.6 < Nat.7
    lt_trans(Nat.4, Nat.6, Nat.7)
    Nat.4 < Nat.7
    lt_suc(Nat.7)
    Nat.7 < Nat.8
    lt_trans(Nat.4, Nat.7, Nat.8)
    Nat.4 < Nat.8
    lt_suc(Nat.8)
    Nat.8 < Nat.9
    lt_trans(Nat.4, Nat.8, Nat.9)
    Nat.4 < Nat.9
    lt_suc(Nat.9)
    Nat.9 < Nat.10
    lt_trans(Nat.4, Nat.9, Nat.10)
    Nat.4 < Nat.10
    lt_suc(Nat.10)
    Nat.10 < Nat.11
    lt_trans(Nat.4, Nat.10, Nat.11)
    Nat.4 < Nat.11
    lt_suc(Nat.11)
    Nat.11 < Nat.11.suc
    Nat.11.suc = Nat.12
    Nat.11 < Nat.12
    lt_trans(Nat.4, Nat.11, Nat.12)
    Nat.4 < Nat.12
}

theorem lt_cancel_pos_mul(a: Real, b: Real, c: Real) {
    a * c < b * c and c.is_positive implies a < b
} by {
    if a * c < b * c and c.is_positive {
        c > Real.0
        div_lt_div_pos(a * c, b * c, c)
        (a * c) / c < (b * c) / c
        c != Real.0
        from real.real_field import div_mul_cancel_left
        div_mul_cancel_left(c, a)
        (c * a) / c = a
        a * c = c * a
        (a * c) / c = a
        div_mul_cancel_left(c, b)
        (c * b) / c = b
        b * c = c * b
        (b * c) / c = b
        a < b
    }
}

/// The absolute values of the cosine terms at two are strictly decreasing from index one.
theorem cos_two_term_abs_decreasing(n: Nat) {
    n >= Nat.1 implies cos_term(two, n.suc).abs < cos_term(two, n).abs
} by {
    if n >= Nat.1 {
        two > Real.0
        two.is_positive
        not two.is_negative
        two.abs = two
        cos_term_abs_exp(two, n)
        cos_term(two, n).abs = exp_term(two.abs, Nat.2 * n)
        exp_term(two.abs, Nat.2 * n) = exp_term(two, Nat.2 * n)
        cos_term(two, n).abs = exp_term(two, Nat.2 * n)
        cos_term_abs_exp(two, n.suc)
        cos_term(two, n.suc).abs = exp_term(two.abs, Nat.2 * n.suc)
        exp_term(two.abs, Nat.2 * n.suc) = exp_term(two, Nat.2 * n.suc)
        cos_term(two, n.suc).abs = exp_term(two, Nat.2 * n.suc)
        let f1 = Real.from_rat(Rat.from_nat((Nat.2 * n).suc))
        let f2 = Real.from_rat(Rat.from_nat((Nat.2 * n).suc.suc))
        exp_term_mul_recurrence(two, Nat.2 * n)
        exp_term(two, (Nat.2 * n).suc) * f1 = two * exp_term(two, Nat.2 * n)
        exp_term_mul_recurrence(two, (Nat.2 * n).suc)
        exp_term(two, (Nat.2 * n).suc.suc) * f2 = two * exp_term(two, (Nat.2 * n).suc)
        two_mul_suc(n)
        Nat.2 * n.suc = (Nat.2 * n).suc.suc
        exp_term(two, Nat.2 * n.suc) = exp_term(two, (Nat.2 * n).suc.suc)
        exp_term(two, Nat.2 * n.suc) * f2 = two * exp_term(two, (Nat.2 * n).suc)
        exp_term(two, Nat.2 * n.suc) * f2 * f1 = (two * exp_term(two, (Nat.2 * n).suc)) * f1
        (two * exp_term(two, (Nat.2 * n).suc)) * f1 = two * (exp_term(two, (Nat.2 * n).suc) * f1)
        exp_term(two, (Nat.2 * n).suc) * f1 = two * exp_term(two, Nat.2 * n)
        two * (exp_term(two, (Nat.2 * n).suc) * f1) = two * (two * exp_term(two, Nat.2 * n))
        two * (two * exp_term(two, Nat.2 * n)) = (two * two) * exp_term(two, Nat.2 * n)
        Real.from_rat(Rat.from_nat(Nat.2)) = two
        two = Real.from_rat(Rat.from_nat(Nat.2))
        two * two = Real.from_rat(Rat.from_nat(Nat.2)) * Real.from_rat(Rat.from_nat(Nat.2))
        mul_from_rat(Rat.from_nat(Nat.2), Rat.from_nat(Nat.2))
        Real.from_rat(Rat.from_nat(Nat.2)) * Real.from_rat(Rat.from_nat(Nat.2)) =
            Real.from_rat(Rat.from_nat(Nat.2) * Rat.from_nat(Nat.2))
        Rat.from_nat(Nat.2) * Rat.from_nat(Nat.2) = Rat.from_nat(Nat.2 * Nat.2)
        Nat.2 * Nat.2 = Nat.4
        Rat.from_nat(Nat.2) * Rat.from_nat(Nat.2) = Rat.from_nat(Nat.4)
        Real.from_rat(Rat.from_nat(Nat.2)) * Real.from_rat(Rat.from_nat(Nat.2)) =
            Real.from_rat(Rat.from_nat(Nat.4))
        two * two = Real.from_rat(Rat.from_nat(Nat.4))
        (two * two) * exp_term(two, Nat.2 * n) = Real.from_rat(Rat.from_nat(Nat.4)) * exp_term(two, Nat.2 * n)
        exp_term(two, Nat.2 * n.suc) * f2 * f1 = Real.from_rat(Rat.from_nat(Nat.4)) * exp_term(two, Nat.2 * n)
        exp_term(two, Nat.2 * n.suc) * (f2 * f1) = Real.from_rat(Rat.from_nat(Nat.4)) * exp_term(two, Nat.2 * n)
        f2 * f1 = f1 * f2
        exp_term(two, Nat.2 * n.suc) * (f1 * f2) = Real.from_rat(Rat.from_nat(Nat.4)) * exp_term(two, Nat.2 * n)
        f1 = Real.from_rat(Rat.from_nat(Nat.2 * n + Nat.1))
        f2 = Real.from_rat(Rat.from_nat(Nat.2 * n + Nat.2))
        f1 * f2 = Real.from_rat(Rat.from_nat(Nat.2 * n + Nat.1)) * Real.from_rat(Rat.from_nat(Nat.2 * n + Nat.2))
        mul_from_rat(Rat.from_nat(Nat.2 * n + Nat.1), Rat.from_nat(Nat.2 * n + Nat.2))
        Real.from_rat(Rat.from_nat(Nat.2 * n + Nat.1)) * Real.from_rat(Rat.from_nat(Nat.2 * n + Nat.2)) =
            Real.from_rat(Rat.from_nat(Nat.2 * n + Nat.1) * Rat.from_nat(Nat.2 * n + Nat.2))
        rat_from_nat_mul(Nat.2 * n + Nat.1, Nat.2 * n + Nat.2)
        Rat.from_nat(Nat.2 * n + Nat.1) * Rat.from_nat(Nat.2 * n + Nat.2) =
            Rat.from_nat((Nat.2 * n + Nat.1) * (Nat.2 * n + Nat.2))
        f1 * f2 = Real.from_rat(Rat.from_nat((Nat.2 * n + Nat.1) * (Nat.2 * n + Nat.2)))
        n >= Nat.1
        Nat.1 <= n
        lte_mul_both(Nat.2, Nat.1, n)
        Nat.2 * Nat.1 <= Nat.2 * n
        Nat.2 * Nat.1 = Nat.2
        Nat.2 <= Nat.2 * n
        lte_add_left(Nat.2, Nat.2, Nat.2 * n)
        Nat.2 + Nat.2 <= Nat.2 + Nat.2 * n
        Nat.2 + Nat.2 * n = Nat.2 * n + Nat.2
        Nat.2 + Nat.2 <= Nat.2 * n + Nat.2
        Nat.2 + Nat.2 = Nat.4
        Nat.4 <= Nat.2 * n + Nat.2
        lte_add_left(Nat.1, Nat.2, Nat.2 * n)
        Nat.1 + Nat.2 <= Nat.1 + Nat.2 * n
        Nat.1 + Nat.2 * n = Nat.2 * n + Nat.1
        Nat.2 + Nat.1 = Nat.1 + Nat.2
        Nat.2 + Nat.1 <= Nat.2 * n + Nat.1
        Nat.2 + Nat.1 = Nat.3
        Nat.3 <= Nat.2 * n + Nat.1
        lte_mul_both(Nat.3, Nat.4, Nat.2 * n + Nat.2)
        Nat.3 * Nat.4 <= Nat.3 * (Nat.2 * n + Nat.2)
        Nat.3 * Nat.4 = Nat.12
        Nat.12 <= Nat.3 * (Nat.2 * n + Nat.2)
        mul_comm(Nat.3, Nat.2 * n + Nat.2)
        Nat.3 * (Nat.2 * n + Nat.2) = (Nat.2 * n + Nat.2) * Nat.3
        Nat.12 <= (Nat.2 * n + Nat.2) * Nat.3
        lte_mul_both(Nat.2 * n + Nat.2, Nat.3, Nat.2 * n + Nat.1)
        (Nat.2 * n + Nat.2) * Nat.3 <= (Nat.2 * n + Nat.2) * (Nat.2 * n + Nat.1)
        lte_trans(Nat.12, (Nat.2 * n + Nat.2) * Nat.3, (Nat.2 * n + Nat.2) * (Nat.2 * n + Nat.1))
        Nat.12 <= (Nat.2 * n + Nat.2) * (Nat.2 * n + Nat.1)
        mul_comm(Nat.2 * n + Nat.1, Nat.2 * n + Nat.2)
        (Nat.2 * n + Nat.1) * (Nat.2 * n + Nat.2) = (Nat.2 * n + Nat.2) * (Nat.2 * n + Nat.1)
        lte_trans(Nat.12, (Nat.2 * n + Nat.2) * (Nat.2 * n + Nat.1), (Nat.2 * n + Nat.1) * (Nat.2 * n + Nat.2))
        Nat.12 <= (Nat.2 * n + Nat.1) * (Nat.2 * n + Nat.2)
        four_lt_twelve
        Nat.4 < Nat.12
        lt_of_lt_of_lte(Nat.4, Nat.12, (Nat.2 * n + Nat.1) * (Nat.2 * n + Nat.2))
        Nat.4 < (Nat.2 * n + Nat.1) * (Nat.2 * n + Nat.2)
        nat_lt_imp_rat_lt(Nat.4, (Nat.2 * n + Nat.1) * (Nat.2 * n + Nat.2))
        Rat.from_nat(Nat.4) < Rat.from_nat((Nat.2 * n + Nat.1) * (Nat.2 * n + Nat.2))
        Rat.from_nat(Nat.4) = Rat.4
        Rat.4 < Rat.from_nat((Nat.2 * n + Nat.1) * (Nat.2 * n + Nat.2))
        from_rat_maintains_lt(Rat.4, Rat.from_nat((Nat.2 * n + Nat.1) * (Nat.2 * n + Nat.2)))
        Real.from_rat(Rat.4) < Real.from_rat(Rat.from_nat((Nat.2 * n + Nat.1) * (Nat.2 * n + Nat.2)))
        Real.from_rat(Rat.4) = Real.from_rat(Rat.from_nat(Nat.4))
        Real.from_rat(Rat.from_nat(Nat.4)) < Real.from_rat(Rat.from_nat((Nat.2 * n + Nat.1) * (Nat.2 * n + Nat.2)))
        Real.from_rat(Rat.from_nat(Nat.4)) < f1 * f2
        two > Real.0
        exp_term_pos(two, Nat.2 * n)
        exp_term(two, Nat.2 * n) > Real.0
        exp_term(two, Nat.2 * n).is_positive
        lt_mul_pos_left(Real.from_rat(Rat.from_nat(Nat.4)), f1 * f2, exp_term(two, Nat.2 * n))
        Real.from_rat(Rat.from_nat(Nat.4)) * exp_term(two, Nat.2 * n) < (f1 * f2) * exp_term(two, Nat.2 * n)
        exp_term(two, Nat.2 * n.suc) * (f1 * f2) < (f1 * f2) * exp_term(two, Nat.2 * n)
        suc_pos(Nat.2 * n)
        f1 > Real.0
        f1.is_positive
        suc_pos((Nat.2 * n).suc)
        f2 > Real.0
        f2.is_positive
        mul_pos_pos(f1, f2)
        (f1 * f2).is_positive
        f1 * f2 > Real.0
        lt_cancel_pos_mul(exp_term(two, Nat.2 * n.suc), exp_term(two, Nat.2 * n), f1 * f2)
        exp_term(two, Nat.2 * n.suc) < exp_term(two, Nat.2 * n)
        cos_term(two, n.suc).abs < cos_term(two, n).abs
    }
}


/// Two times two is four.
theorem two_mul_two_eq_four {
    two * two = Real.from_rat(Rat.from_nat(Nat.4))
} by {
    Real.from_rat(Rat.from_nat(Nat.2)) = two
    two = Real.from_rat(Rat.from_nat(Nat.2))
    two * two = Real.from_rat(Rat.from_nat(Nat.2)) * Real.from_rat(Rat.from_nat(Nat.2))
    mul_from_rat(Rat.from_nat(Nat.2), Rat.from_nat(Nat.2))
    Real.from_rat(Rat.from_nat(Nat.2)) * Real.from_rat(Rat.from_nat(Nat.2)) =
        Real.from_rat(Rat.from_nat(Nat.2) * Rat.from_nat(Nat.2))
    rat_from_nat_mul(Nat.2, Nat.2)
    Rat.from_nat(Nat.2) * Rat.from_nat(Nat.2) = Rat.from_nat(Nat.2 * Nat.2)
    Nat.2 * Nat.2 = Nat.4
    Rat.from_nat(Nat.2) * Rat.from_nat(Nat.2) = Rat.from_nat(Nat.4)
    Real.from_rat(Rat.from_nat(Nat.2)) * Real.from_rat(Rat.from_nat(Nat.2)) =
        Real.from_rat(Rat.from_nat(Nat.4))
    two * two = Real.from_rat(Rat.from_nat(Nat.4))
}

/// One plus negative two is negative one.
theorem one_plus_neg_two {
    Real.1 + -two = -Real.1
} by {
    two = Real.1 + Real.1
    Real.1 + -two = Real.1 + -(Real.1 + Real.1)
    Real.1 + -(Real.1 + Real.1) = Real.1 + (-Real.1 + -Real.1)
    add_assoc(Real.1, -Real.1, -Real.1)
    Real.1 + (-Real.1 + -Real.1) = (Real.1 + -Real.1) + -Real.1
    Real.1 + -Real.1 = Real.0
    (Real.1 + -Real.1) + -Real.1 = Real.0 + -Real.1
    Real.0 + -Real.1 = -Real.1
    Real.1 + -(Real.1 + Real.1) = -Real.1
    Real.1 + -two = -Real.1
}

/// Two to the fourth is sixteen.
theorem two_pow_four_eq_sixteen {
    two.pow(Nat.4) = Real.from_rat(Rat.from_nat(Nat.16))
} by {
    pow_suc(two, Nat.1)
    two.pow(Nat.2) = two * two
    two_mul_two_eq_four
    two.pow(Nat.2) = Real.from_rat(Rat.from_nat(Nat.4))
    pow_add(two, Nat.2, Nat.2)
    two.pow(Nat.2 + Nat.2) = two.pow(Nat.2) * two.pow(Nat.2)
    Nat.2 + Nat.2 = Nat.4
    two.pow(Nat.4) = two.pow(Nat.2) * two.pow(Nat.2)
    two.pow(Nat.4) = Real.from_rat(Rat.from_nat(Nat.4)) * Real.from_rat(Rat.from_nat(Nat.4))
    mul_from_rat(Rat.from_nat(Nat.4), Rat.from_nat(Nat.4))
    Real.from_rat(Rat.from_nat(Nat.4)) * Real.from_rat(Rat.from_nat(Nat.4)) =
        Real.from_rat(Rat.from_nat(Nat.4) * Rat.from_nat(Nat.4))
    rat_from_nat_mul(Nat.4, Nat.4)
    Rat.from_nat(Nat.4) * Rat.from_nat(Nat.4) = Rat.from_nat(Nat.4 * Nat.4)
    Nat.4 * Nat.4 = Nat.16
    Rat.from_nat(Nat.4) * Rat.from_nat(Nat.4) = Rat.from_nat(Nat.16)
    Real.from_rat(Rat.from_nat(Nat.4)) * Real.from_rat(Rat.from_nat(Nat.4)) =
        Real.from_rat(Rat.from_nat(Nat.16))
    two.pow(Nat.4) = Real.from_rat(Rat.from_nat(Nat.16))
}


/// The tail of the cosine series at two from index two.
define cos_two_tail(j: Nat) -> Real {
    cos_term(two, j + Nat.2)
}


/// The cosine terms at two with odd index are negative.
theorem cos_two_term_odd_neg(n: Nat) {
    cos_term(two, Nat.2 * n + Nat.1) = -cos_term(two, Nat.2 * n + Nat.1).abs
} by {
    cos_term(two, Nat.2 * n + Nat.1) = alternating_sign[Real](Nat.2 * n + Nat.1) * two.pow(Nat.2 * (Nat.2 * n + Nat.1)) / Real.from_rat(Rat.from_nat((Nat.2 * (Nat.2 * n + Nat.1)).factorial))
    cos_term_abs(two, Nat.2 * n + Nat.1)
    cos_term(two, Nat.2 * n + Nat.1).abs = two.abs.pow(Nat.2 * (Nat.2 * n + Nat.1)) / Real.from_rat(Rat.from_nat((Nat.2 * (Nat.2 * n + Nat.1)).factorial))
    two > Real.0
    two.is_positive
    not two.is_negative
    two.abs = two
    two.abs.pow(Nat.2 * (Nat.2 * n + Nat.1)) = two.pow(Nat.2 * (Nat.2 * n + Nat.1))
    cos_term(two, Nat.2 * n + Nat.1).abs = two.pow(Nat.2 * (Nat.2 * n + Nat.1)) / Real.from_rat(Rat.from_nat((Nat.2 * (Nat.2 * n + Nat.1)).factorial))
    alternating_sign_double_suc(n)
    alternating_sign[Real](Nat.2 * n + Nat.1) = -Real.1
    alternating_sign[Real](Nat.2 * n + Nat.1) * two.pow(Nat.2 * (Nat.2 * n + Nat.1)) / Real.from_rat(Rat.from_nat((Nat.2 * (Nat.2 * n + Nat.1)).factorial)) =
        -Real.1 * (two.pow(Nat.2 * (Nat.2 * n + Nat.1)) / Real.from_rat(Rat.from_nat((Nat.2 * (Nat.2 * n + Nat.1)).factorial)))
    -Real.1 * (two.pow(Nat.2 * (Nat.2 * n + Nat.1)) / Real.from_rat(Rat.from_nat((Nat.2 * (Nat.2 * n + Nat.1)).factorial))) =
        -(two.pow(Nat.2 * (Nat.2 * n + Nat.1)) / Real.from_rat(Rat.from_nat((Nat.2 * (Nat.2 * n + Nat.1)).factorial)))
    cos_term(two, Nat.2 * n + Nat.1) = -(two.pow(Nat.2 * (Nat.2 * n + Nat.1)) / Real.from_rat(Rat.from_nat((Nat.2 * (Nat.2 * n + Nat.1)).factorial)))
    cos_term(two, Nat.2 * n + Nat.1) = -cos_term(two, Nat.2 * n + Nat.1).abs
}

/// The cosine terms at two with even index are positive.
theorem cos_two_term_even_pos(n: Nat) {
    cos_term(two, Nat.2 * n) = cos_term(two, Nat.2 * n).abs
} by {
    cos_term(two, Nat.2 * n) = alternating_sign[Real](Nat.2 * n) * two.pow(Nat.2 * (Nat.2 * n)) / Real.from_rat(Rat.from_nat((Nat.2 * (Nat.2 * n)).factorial))
    cos_term_abs(two, Nat.2 * n)
    cos_term(two, Nat.2 * n).abs = two.abs.pow(Nat.2 * (Nat.2 * n)) / Real.from_rat(Rat.from_nat((Nat.2 * (Nat.2 * n)).factorial))
    two > Real.0
    two.is_positive
    not two.is_negative
    two.abs = two
    two.abs.pow(Nat.2 * (Nat.2 * n)) = two.pow(Nat.2 * (Nat.2 * n))
    cos_term(two, Nat.2 * n).abs = two.pow(Nat.2 * (Nat.2 * n)) / Real.from_rat(Rat.from_nat((Nat.2 * (Nat.2 * n)).factorial))
    alternating_sign_double(n)
    alternating_sign[Real](Nat.2 * n) = Real.1
    alternating_sign[Real](Nat.2 * n) * two.pow(Nat.2 * (Nat.2 * n)) / Real.from_rat(Rat.from_nat((Nat.2 * (Nat.2 * n)).factorial)) =
        Real.1 * (two.pow(Nat.2 * (Nat.2 * n)) / Real.from_rat(Rat.from_nat((Nat.2 * (Nat.2 * n)).factorial)))
    Real.1 * (two.pow(Nat.2 * (Nat.2 * n)) / Real.from_rat(Rat.from_nat((Nat.2 * (Nat.2 * n)).factorial))) =
        two.pow(Nat.2 * (Nat.2 * n)) / Real.from_rat(Rat.from_nat((Nat.2 * (Nat.2 * n)).factorial))
    cos_term(two, Nat.2 * n) = two.pow(Nat.2 * (Nat.2 * n)) / Real.from_rat(Rat.from_nat((Nat.2 * (Nat.2 * n)).factorial))
    cos_term(two, Nat.2 * n) = cos_term(two, Nat.2 * n).abs
}

/// A pair of consecutive tail terms starting at an odd index has nonpositive sum.
theorem cos_two_tail_odd_pair_nonpos(m: Nat) {
    cos_two_tail(Nat.2 * m + Nat.1) + cos_two_tail(Nat.2 * m + Nat.2) <= Real.0
} by {
    cos_two_tail(Nat.2 * m + Nat.1) = cos_term(two, Nat.2 * m + Nat.1 + Nat.2)
    cos_two_tail_index_add_one(m)
    Nat.2 * m + Nat.1 + Nat.2 = Nat.2 * m + Nat.3
    cos_two_tail(Nat.2 * m + Nat.1) = cos_term(two, Nat.2 * m + Nat.3)
    cos_two_tail(Nat.2 * m + Nat.2) = cos_term(two, Nat.2 * m + Nat.2 + Nat.2)
    cos_two_tail_index_add_two(m)
    Nat.2 * m + Nat.2 + Nat.2 = Nat.2 * m + Nat.4
    cos_two_tail(Nat.2 * m + Nat.2) = cos_term(two, Nat.2 * m + Nat.4)
    cos_two_tail_pair_index_odd(m)
    Nat.2 * m.suc + Nat.1 = Nat.2 * m + Nat.3
    cos_two_tail_double_suc_index(m)
    Nat.2 * m.suc.suc = Nat.2 * m + Nat.4
    cos_two_term_odd_neg(m.suc)
    cos_term(two, Nat.2 * m.suc + Nat.1) = -cos_term(two, Nat.2 * m.suc + Nat.1).abs
    cos_two_tail(Nat.2 * m + Nat.1) = -cos_term(two, Nat.2 * m.suc + Nat.1).abs
    cos_two_tail(Nat.2 * m + Nat.1) = -cos_term(two, Nat.2 * m + Nat.3).abs
    cos_two_term_even_pos(m.suc.suc)
    cos_term(two, Nat.2 * m.suc.suc) = cos_term(two, Nat.2 * m.suc.suc).abs
    cos_two_tail(Nat.2 * m + Nat.2) = cos_term(two, Nat.2 * m + Nat.4).abs
    cos_two_tail_odd_ge_one(m)
    Nat.1 <= Nat.2 * m + Nat.3
    cos_two_term_abs_decreasing(Nat.2 * m + Nat.3)
    cos_term(two, (Nat.2 * m + Nat.3).suc).abs < cos_term(two, Nat.2 * m + Nat.3).abs
    cos_two_tail_odd_suc(m)
    (Nat.2 * m + Nat.3).suc = Nat.2 * m + Nat.4
    cos_term(two, Nat.2 * m + Nat.4).abs < cos_term(two, Nat.2 * m + Nat.3).abs
    cos_two_tail(Nat.2 * m + Nat.1) + cos_two_tail(Nat.2 * m + Nat.2) =
        -cos_term(two, Nat.2 * m + Nat.3).abs + cos_term(two, Nat.2 * m + Nat.4).abs
    lt_add_right(cos_term(two, Nat.2 * m + Nat.4).abs, cos_term(two, Nat.2 * m + Nat.3).abs, -cos_term(two, Nat.2 * m + Nat.3).abs)
    cos_term(two, Nat.2 * m + Nat.4).abs + (-cos_term(two, Nat.2 * m + Nat.3).abs) < cos_term(two, Nat.2 * m + Nat.3).abs + (-cos_term(two, Nat.2 * m + Nat.3).abs)
    cos_term(two, Nat.2 * m + Nat.3).abs + (-cos_term(two, Nat.2 * m + Nat.3).abs) = Real.0
    cos_term(two, Nat.2 * m + Nat.4).abs + (-cos_term(two, Nat.2 * m + Nat.3).abs) < Real.0
    -cos_term(two, Nat.2 * m + Nat.3).abs + cos_term(two, Nat.2 * m + Nat.4).abs =
        cos_term(two, Nat.2 * m + Nat.4).abs + (-cos_term(two, Nat.2 * m + Nat.3).abs)
    -cos_term(two, Nat.2 * m + Nat.3).abs + cos_term(two, Nat.2 * m + Nat.4).abs < Real.0
    cos_two_tail(Nat.2 * m + Nat.1) + cos_two_tail(Nat.2 * m + Nat.2) < Real.0
    cos_two_tail(Nat.2 * m + Nat.1) + cos_two_tail(Nat.2 * m + Nat.2) <= Real.0
}

/// The odd-indexed partial sums of the tail decrease.
theorem cos_two_tail_odd_decreasing(k: Nat) {
    partial(cos_two_tail, Nat.2 * k.suc + Nat.3) <= partial(cos_two_tail, Nat.2 * k + Nat.3)
} by {
    partial_split_last(cos_two_tail, Nat.2 * k + Nat.3)
    partial(cos_two_tail, (Nat.2 * k + Nat.3).suc) = partial(cos_two_tail, Nat.2 * k + Nat.3) + cos_two_tail(Nat.2 * k + Nat.3)
    partial_split_last(cos_two_tail, (Nat.2 * k + Nat.3).suc)
    partial(cos_two_tail, (Nat.2 * k + Nat.3).suc.suc) = partial(cos_two_tail, (Nat.2 * k + Nat.3).suc) + cos_two_tail((Nat.2 * k + Nat.3).suc)
    cos_two_tail_odd_partial_index(k)
    Nat.2 * k.suc + Nat.3 = (Nat.2 * k + Nat.3).suc.suc
    partial(cos_two_tail, Nat.2 * k.suc + Nat.3) =
        partial(cos_two_tail, Nat.2 * k + Nat.3) + cos_two_tail(Nat.2 * k + Nat.3) + cos_two_tail((Nat.2 * k + Nat.3).suc)
    cos_two_tail_odd_suc(k)
    (Nat.2 * k + Nat.3).suc = Nat.2 * k + Nat.4
    partial(cos_two_tail, Nat.2 * k.suc + Nat.3) =
        partial(cos_two_tail, Nat.2 * k + Nat.3) + cos_two_tail(Nat.2 * k + Nat.3) + cos_two_tail(Nat.2 * k + Nat.4)
    add_assoc(partial(cos_two_tail, Nat.2 * k + Nat.3), cos_two_tail(Nat.2 * k + Nat.3), cos_two_tail(Nat.2 * k + Nat.4))
    partial(cos_two_tail, Nat.2 * k + Nat.3) + cos_two_tail(Nat.2 * k + Nat.3) + cos_two_tail(Nat.2 * k + Nat.4) =
        partial(cos_two_tail, Nat.2 * k + Nat.3) + (cos_two_tail(Nat.2 * k + Nat.3) + cos_two_tail(Nat.2 * k + Nat.4))
    partial(cos_two_tail, Nat.2 * k.suc + Nat.3) =
        partial(cos_two_tail, Nat.2 * k + Nat.3) + (cos_two_tail(Nat.2 * k + Nat.3) + cos_two_tail(Nat.2 * k + Nat.4))
    cos_two_tail_odd_pair_nonpos(k.suc)
    cos_two_tail(Nat.2 * k.suc + Nat.1) + cos_two_tail(Nat.2 * k.suc + Nat.2) <= Real.0
    cos_two_tail_pair_index_odd(k.suc)
    Nat.2 * k.suc + Nat.1 = Nat.2 * k + Nat.3
    cos_two_tail_pair_index_even(k.suc)
    Nat.2 * k.suc + Nat.2 = Nat.2 * k + Nat.4
    cos_two_tail(Nat.2 * k + Nat.3) + cos_two_tail(Nat.2 * k + Nat.4) <= Real.0
    lte_add_right(cos_two_tail(Nat.2 * k + Nat.3) + cos_two_tail(Nat.2 * k + Nat.4), Real.0, partial(cos_two_tail, Nat.2 * k + Nat.3))
    (cos_two_tail(Nat.2 * k + Nat.3) + cos_two_tail(Nat.2 * k + Nat.4)) + partial(cos_two_tail, Nat.2 * k + Nat.3) <= Real.0 + partial(cos_two_tail, Nat.2 * k + Nat.3)
    partial(cos_two_tail, Nat.2 * k + Nat.3) + (cos_two_tail(Nat.2 * k + Nat.3) + cos_two_tail(Nat.2 * k + Nat.4)) <= partial(cos_two_tail, Nat.2 * k + Nat.3)
    partial(cos_two_tail, Nat.2 * k.suc + Nat.3) <= partial(cos_two_tail, Nat.2 * k + Nat.3)
}

/// The odd-indexed partial sums of the tail are bounded by the third partial sum.
theorem cos_two_tail_odd_le_three(k: Nat) {
    partial(cos_two_tail, Nat.2 * k + Nat.3) <= partial(cos_two_tail, Nat.3)
} by {
    define p(j: Nat) -> Bool {
        partial(cos_two_tail, Nat.2 * j + Nat.3) <= partial(cos_two_tail, Nat.3)
    }
    lte_refl(partial(cos_two_tail, Nat.3))
    partial(cos_two_tail, Nat.3) <= partial(cos_two_tail, Nat.3)
    p(Nat.0)
    forall(j: Nat) {
        if p(j) {
            cos_two_tail_odd_decreasing(j)
            partial(cos_two_tail, Nat.2 * j.suc + Nat.3) <= partial(cos_two_tail, Nat.2 * j + Nat.3)
            p(j)
            partial(cos_two_tail, Nat.2 * j + Nat.3) <= partial(cos_two_tail, Nat.3)
            lte_trans(partial(cos_two_tail, Nat.2 * j.suc + Nat.3), partial(cos_two_tail, Nat.2 * j + Nat.3), partial(cos_two_tail, Nat.3))
            partial(cos_two_tail, Nat.2 * j.suc + Nat.3) <= partial(cos_two_tail, Nat.3)
            p(j.suc)
        }
    }
    p(Nat.0) and forall(j: Nat) { p(j) implies p(j.suc) }
    Nat.induction(p)
    p(k)
}

/// The tail series at two converges.
theorem cos_two_tail_converges {
    converges(partial(cos_two_tail))
} by {
    cos_term_abs_converges(two)
    absolutely_converges(cos_term(two))
    absolutely_converges_imp_converges(cos_term(two))
    converges(partial(cos_term(two)))
    tail_partial_converges(cos_term(two), Nat.2)
    converges_to(partial(tail(cos_term(two), Nat.2)), limit(partial(cos_term(two))) - partial(cos_term(two), Nat.2))
    converges_to_imp_converges(partial(tail(cos_term(two), Nat.2)), limit(partial(cos_term(two))) - partial(cos_term(two), Nat.2))
    converges(partial(tail(cos_term(two), Nat.2)))
    forall(j: Nat) {
        tail(cos_term(two), Nat.2, j) = cos_term(two, Nat.2 + j)
        cos_two_tail(j) = cos_term(two, j + Nat.2)
        Nat.2 + j = j + Nat.2
        tail(cos_term(two), Nat.2, j) = cos_two_tail(j)
    }
    tail(cos_term(two), Nat.2) = cos_two_tail
    partial(tail(cos_term(two), Nat.2)) = partial(cos_two_tail)
    converges(partial(cos_two_tail))
}

/// The limit of the tail series at two is cosine of two minus its second partial sum.
theorem cos_two_tail_limit_eq {
    limit(partial(cos_two_tail)) = two.cos - partial(cos_term(two), Nat.2)
} by {
    cos_two_tail_converges
    converges(partial(cos_two_tail))
    converges_imp_converges_to(partial(cos_two_tail))
    converges_to(partial(cos_two_tail), limit(partial(cos_two_tail)))
    cos_term_abs_converges(two)
    absolutely_converges(cos_term(two))
    absolutely_converges_imp_converges(cos_term(two))
    converges(partial(cos_term(two)))
    tail_partial_converges(cos_term(two), Nat.2)
    converges_to(partial(tail(cos_term(two), Nat.2)), limit(partial(cos_term(two))) - partial(cos_term(two), Nat.2))
    forall(j: Nat) {
        tail(cos_term(two), Nat.2, j) = cos_term(two, Nat.2 + j)
        cos_two_tail(j) = cos_term(two, j + Nat.2)
        Nat.2 + j = j + Nat.2
        tail(cos_term(two), Nat.2, j) = cos_two_tail(j)
    }
    tail(cos_term(two), Nat.2) = cos_two_tail
    partial(tail(cos_term(two), Nat.2)) = partial(cos_two_tail)
    converges_to(partial(cos_two_tail), limit(partial(cos_term(two))) - partial(cos_term(two), Nat.2))
    converges_to_unique(partial(cos_two_tail), limit(partial(cos_two_tail)), limit(partial(cos_term(two))) - partial(cos_term(two), Nat.2))
    limit(partial(cos_two_tail)) = limit(partial(cos_term(two))) - partial(cos_term(two), Nat.2)
    two.cos = limit(partial(cos_term(two)))
    limit(partial(cos_two_tail)) = two.cos - partial(cos_term(two), Nat.2)
}

/// The index map that selects the odd-indexed tail partial sums.
define odd_shift_index(k: Nat) -> Nat {
    Nat.2 * k + Nat.3
}

/// The odd-indexed partial sums of the tail series at two.
define cos_two_tail_odd_partial(k: Nat) -> Real {
    partial(cos_two_tail, odd_shift_index(k))
}

/// The map selecting odd indices is a subsequence index.
theorem odd_shift_subsequence_index {
    is_subsequence_index(odd_shift_index)
} by {
    forall(i: Nat, j: Nat) {
        if i <= j {
            lte_mul_both(Nat.2, i, j)
            Nat.2 * i <= Nat.2 * j
            lte_add_left(Nat.3, Nat.2 * i, Nat.2 * j)
            Nat.3 + Nat.2 * i <= Nat.3 + Nat.2 * j
            Nat.3 + Nat.2 * i = Nat.2 * i + Nat.3
            Nat.3 + Nat.2 * j = Nat.2 * j + Nat.3
            Nat.2 * i + Nat.3 <= Nat.2 * j + Nat.3
            odd_shift_index(i) <= odd_shift_index(j)
        }
    }
    is_monotone(odd_shift_index)
    forall(bound: Nat) {
        lt_add_suc(bound, Nat.2)
        bound < bound + Nat.3
        lte_mul_both(bound, Nat.1, Nat.2)
        bound * Nat.1 <= bound * Nat.2
        bound * Nat.1 = bound
        bound * Nat.2 = Nat.2 * bound
        bound <= Nat.2 * bound
        lte_add_left(Nat.3, bound, Nat.2 * bound)
        Nat.3 + bound <= Nat.3 + Nat.2 * bound
        Nat.3 + bound = bound + Nat.3
        Nat.3 + Nat.2 * bound = Nat.2 * bound + Nat.3
        bound + Nat.3 <= Nat.2 * bound + Nat.3
        lt_of_lt_of_lte(bound, bound + Nat.3, Nat.2 * bound + Nat.3)
        bound < Nat.2 * bound + Nat.3
        odd_shift_index(bound) = Nat.2 * bound + Nat.3
        bound < odd_shift_index(bound)
        exists(n: Nat) {
            bound < odd_shift_index(n)
        }
    }
    is_unbounded(odd_shift_index)
    subsequence_index_of_monotone_unbounded(odd_shift_index)
    is_subsequence_index(odd_shift_index)
}

/// The odd-indexed tail partial sums converge to the limit of the tail series.
theorem cos_two_tail_odd_subseq_converges {
    converges_to(cos_two_tail_odd_partial, limit(partial(cos_two_tail)))
} by {
    cos_two_tail_converges
    converges(partial(cos_two_tail))
    odd_shift_subsequence_index
    is_subsequence_index(odd_shift_index)
    converges_subsequence(partial(cos_two_tail), odd_shift_index)
    converges_to(subsequence(partial(cos_two_tail), odd_shift_index), limit(partial(cos_two_tail)))
    forall(k: Nat) {
        subsequence(partial(cos_two_tail), odd_shift_index, k) = partial(cos_two_tail)(odd_shift_index(k))
        partial(cos_two_tail)(odd_shift_index(k)) = partial(cos_two_tail, odd_shift_index(k))
        cos_two_tail_odd_partial(k) = partial(cos_two_tail, odd_shift_index(k))
        subsequence(partial(cos_two_tail), odd_shift_index, k) = cos_two_tail_odd_partial(k)
    }
    subsequence(partial(cos_two_tail), odd_shift_index) = cos_two_tail_odd_partial
    converges_to(cos_two_tail_odd_partial, limit(partial(cos_two_tail)))
}

/// The limit of the tail series at two is at most its third partial sum.
theorem cos_two_tail_limit_le_three {
    limit(partial(cos_two_tail)) <= partial(cos_two_tail, Nat.3)
} by {
    cos_two_tail_odd_subseq_converges
    converges_to(cos_two_tail_odd_partial, limit(partial(cos_two_tail)))
    converges_to_imp_converges(cos_two_tail_odd_partial, limit(partial(cos_two_tail)))
    converges(cos_two_tail_odd_partial)
    const_converges(partial(cos_two_tail, Nat.3))
    converges(const_seq(partial(cos_two_tail, Nat.3)))
    forall(k: Nat) {
        cos_two_tail_odd_le_three(k)
        partial(cos_two_tail, Nat.2 * k + Nat.3) <= partial(cos_two_tail, Nat.3)
        cos_two_tail_odd_partial(k) = partial(cos_two_tail, odd_shift_index(k))
        odd_shift_index(k) = Nat.2 * k + Nat.3
        cos_two_tail_odd_partial(k) = partial(cos_two_tail, Nat.2 * k + Nat.3)
        cos_two_tail_odd_partial(k) <= partial(cos_two_tail, Nat.3)
        const_seq(partial(cos_two_tail, Nat.3), k) = partial(cos_two_tail, Nat.3)
        cos_two_tail_odd_partial(k) <= const_seq(partial(cos_two_tail, Nat.3), k)
    }
    seq_lte(cos_two_tail_odd_partial, const_seq(partial(cos_two_tail, Nat.3)))
    seq_lte_preserves_limit(cos_two_tail_odd_partial, const_seq(partial(cos_two_tail, Nat.3)))
    limit(cos_two_tail_odd_partial) <= limit(const_seq(partial(cos_two_tail, Nat.3)))
    const_limit(partial(cos_two_tail, Nat.3))
    limit(const_seq(partial(cos_two_tail, Nat.3))) = partial(cos_two_tail, Nat.3)
    limit(cos_two_tail_odd_partial) <= partial(cos_two_tail, Nat.3)
    converges_imp_converges_to(cos_two_tail_odd_partial)
    converges_to(cos_two_tail_odd_partial, limit(cos_two_tail_odd_partial))
    converges_to_unique(cos_two_tail_odd_partial, limit(partial(cos_two_tail)), limit(cos_two_tail_odd_partial))
    limit(cos_two_tail_odd_partial) = limit(partial(cos_two_tail))
    limit(partial(cos_two_tail)) <= partial(cos_two_tail, Nat.3)
}

/// The second partial sum of the cosine series at two is negative one.
theorem cos_two_partial_two {
    partial(cos_term(two), Nat.2) = -Real.1
} by {
    partial_split_last(cos_term(two), Nat.1)
    partial(cos_term(two), Nat.2) = partial(cos_term(two), Nat.1) + cos_term(two, Nat.1)
    partial_one(cos_term(two))
    partial(cos_term(two), Nat.1) = cos_term(two, Nat.0)
    cos_term(two, Nat.0) = alternating_sign[Real](Nat.0) * two.pow(Nat.0) / Real.from_rat(Rat.from_nat(Nat.0.factorial))
    alternating_sign_zero[Real]
    alternating_sign[Real](Nat.0) = Real.1
    two.pow(Nat.0) = Real.1
    Nat.0.factorial = Nat.1
    Real.from_rat(Rat.from_nat(Nat.1)) = Real.1
    Real.1 * Real.1 = Real.1
    Real.1 / Real.1 = Real.1
    cos_term(two, Nat.0) = Real.1
    partial(cos_term(two), Nat.1) = Real.1
    cos_term(two, Nat.1) = alternating_sign[Real](Nat.1) * two.pow(Nat.2) / Real.from_rat(Rat.from_nat(Nat.2.factorial))
    alternating_sign_suc[Real](Nat.0)
    alternating_sign[Real](Nat.1) = -alternating_sign[Real](Nat.0)
    alternating_sign[Real](Nat.1) = -Real.1
    two.pow(Nat.2) = two * two
    two_mul_two_eq_four
    two * two = Real.from_rat(Rat.from_nat(Nat.4))
    Nat.2.factorial = Nat.2
    Real.from_rat(Rat.from_nat(Nat.2)) = two
    Real.from_rat(Rat.from_nat(Nat.4)) / Real.from_rat(Rat.from_nat(Nat.2)) = two
    cos_term(two, Nat.1) = -Real.1 * (Real.from_rat(Rat.from_nat(Nat.4)) / Real.from_rat(Rat.from_nat(Nat.2)))
    -Real.1 * (Real.from_rat(Rat.from_nat(Nat.4)) / Real.from_rat(Rat.from_nat(Nat.2))) = -two
    cos_term(two, Nat.1) = -two
    partial(cos_term(two), Nat.2) = Real.1 + -two
    one_plus_neg_two
    Real.1 + -two = -Real.1
    partial(cos_term(two), Nat.2) = -Real.1
}

/// The first partial sum of the tail series at two is two thirds.
theorem cos_two_tail_partial_one {
    partial(cos_two_tail, Nat.1) = Real.from_rat(Rat.from_nat(Nat.2)) / Real.from_rat(Rat.from_nat(Nat.3))
} by {
    partial_one(cos_two_tail)
    partial(cos_two_tail, Nat.1) = cos_two_tail(Nat.0)
    cos_two_tail(Nat.0) = cos_term(two, Nat.2)
    cos_two_term_even_pos(Nat.1)
    cos_term(two, Nat.2) = cos_term(two, Nat.2).abs
    cos_term_abs(two, Nat.2)
    cos_term(two, Nat.2).abs = two.abs.pow(Nat.4) / Real.from_rat(Rat.from_nat(Nat.4.factorial))
    two > Real.0
    two.is_positive
    not two.is_negative
    two.abs = two
    two.abs.pow(Nat.4) = two.pow(Nat.4)
    Nat.4.factorial = Nat.24
    two_pow_four_eq_sixteen
    two.pow(Nat.4) = Real.from_rat(Rat.from_nat(Nat.16))
    cos_term(two, Nat.2) = Real.from_rat(Rat.from_nat(Nat.16)) / Real.from_rat(Rat.from_nat(Nat.24))
    mul_from_rat(Rat.from_nat(Nat.2), Rat.from_nat(Nat.8))
    Real.from_rat(Rat.from_nat(Nat.2)) * Real.from_rat(Rat.from_nat(Nat.8)) =
        Real.from_rat(Rat.from_nat(Nat.2) * Rat.from_nat(Nat.8))
    rat_from_nat_mul(Nat.2, Nat.8)
    Rat.from_nat(Nat.2) * Rat.from_nat(Nat.8) = Rat.from_nat(Nat.2 * Nat.8)
    nat_mul_2_8
    Nat.2 * Nat.8 = Nat.16
    Rat.from_nat(Nat.2) * Rat.from_nat(Nat.8) = Rat.from_nat(Nat.16)
    Real.from_rat(Rat.from_nat(Nat.2)) * Real.from_rat(Rat.from_nat(Nat.8)) =
        Real.from_rat(Rat.from_nat(Nat.16))
    mul_from_rat(Rat.from_nat(Nat.3), Rat.from_nat(Nat.8))
    Real.from_rat(Rat.from_nat(Nat.3)) * Real.from_rat(Rat.from_nat(Nat.8)) =
        Real.from_rat(Rat.from_nat(Nat.3) * Rat.from_nat(Nat.8))
    rat_from_nat_mul(Nat.3, Nat.8)
    Rat.from_nat(Nat.3) * Rat.from_nat(Nat.8) = Rat.from_nat(Nat.3 * Nat.8)
    nat_mul_3_8
    Nat.3 * Nat.8 = Nat.24
    Rat.from_nat(Nat.3) * Rat.from_nat(Nat.8) = Rat.from_nat(Nat.24)
    Real.from_rat(Rat.from_nat(Nat.3)) * Real.from_rat(Rat.from_nat(Nat.8)) =
        Real.from_rat(Rat.from_nat(Nat.24))
    from real.real_field import div_cancel_common
    Real.from_rat(Rat.from_nat(Nat.16)) / Real.from_rat(Rat.from_nat(Nat.24)) =
        (Real.from_rat(Rat.from_nat(Nat.2)) * Real.from_rat(Rat.from_nat(Nat.8))) /
        (Real.from_rat(Rat.from_nat(Nat.3)) * Real.from_rat(Rat.from_nat(Nat.8)))
    Real.from_rat(Rat.from_nat(Nat.8)) > Real.0
    Real.from_rat(Rat.from_nat(Nat.8)) != Real.0
    Real.from_rat(Rat.from_nat(Nat.3)) > Real.0
    Real.from_rat(Rat.from_nat(Nat.3)) != Real.0
    div_cancel_common(Real.from_rat(Rat.from_nat(Nat.2)), Real.from_rat(Rat.from_nat(Nat.8)), Real.from_rat(Rat.from_nat(Nat.3)))
    (Real.from_rat(Rat.from_nat(Nat.2)) * Real.from_rat(Rat.from_nat(Nat.8))) /
        (Real.from_rat(Rat.from_nat(Nat.3)) * Real.from_rat(Rat.from_nat(Nat.8))) =
        Real.from_rat(Rat.from_nat(Nat.2)) / Real.from_rat(Rat.from_nat(Nat.3))
    Real.from_rat(Rat.from_nat(Nat.16)) / Real.from_rat(Rat.from_nat(Nat.24)) =
        Real.from_rat(Rat.from_nat(Nat.2)) / Real.from_rat(Rat.from_nat(Nat.3))
    cos_term(two, Nat.2) = Real.from_rat(Rat.from_nat(Nat.2)) / Real.from_rat(Rat.from_nat(Nat.3))
    partial(cos_two_tail, Nat.1) = Real.from_rat(Rat.from_nat(Nat.2)) / Real.from_rat(Rat.from_nat(Nat.3))
}

/// The odd-indexed partial sums of the tail are bounded by the first partial sum.
theorem cos_two_tail_odd_le_one(k: Nat) {
    partial(cos_two_tail, Nat.2 * k + Nat.1) <= partial(cos_two_tail, Nat.1)
} by {
    define p(j: Nat) -> Bool {
        partial(cos_two_tail, Nat.2 * j + Nat.1) <= partial(cos_two_tail, Nat.1)
    }
    lte_refl(partial(cos_two_tail, Nat.1))
    partial(cos_two_tail, Nat.1) <= partial(cos_two_tail, Nat.1)
    p(Nat.0)
    forall(j: Nat) {
        if p(j) {
            // P_{2(j+1)+1} = P_{2j+1} + t(2j+1) + t(2j+2) <= P_{2j+1}
            partial_split_last(cos_two_tail, Nat.2 * j + Nat.1)
            partial(cos_two_tail, (Nat.2 * j + Nat.1).suc) = partial(cos_two_tail, Nat.2 * j + Nat.1) + cos_two_tail(Nat.2 * j + Nat.1)
            partial_split_last(cos_two_tail, (Nat.2 * j + Nat.1).suc)
            partial(cos_two_tail, (Nat.2 * j + Nat.1).suc.suc) = partial(cos_two_tail, (Nat.2 * j + Nat.1).suc) + cos_two_tail((Nat.2 * j + Nat.1).suc)
            partial(cos_two_tail, (Nat.2 * j + Nat.1).suc.suc) =
                partial(cos_two_tail, Nat.2 * j + Nat.1) + cos_two_tail(Nat.2 * j + Nat.1) + cos_two_tail((Nat.2 * j + Nat.1).suc)
            add_assoc(partial(cos_two_tail, Nat.2 * j + Nat.1), cos_two_tail(Nat.2 * j + Nat.1), cos_two_tail((Nat.2 * j + Nat.1).suc))
            partial(cos_two_tail, (Nat.2 * j + Nat.1).suc.suc) =
                partial(cos_two_tail, Nat.2 * j + Nat.1) + (cos_two_tail(Nat.2 * j + Nat.1) + cos_two_tail((Nat.2 * j + Nat.1).suc))
            cos_two_tail_odd_pair_nonpos(j)
            cos_two_tail(Nat.2 * j + Nat.1) + cos_two_tail(Nat.2 * j + Nat.2) <= Real.0
            cos_two_tail_pair_index_even(j)
            Nat.2 * j.suc + Nat.2 = Nat.2 * j + Nat.4
            (Nat.2 * j + Nat.1).suc = Nat.2 * j + Nat.2
            cos_two_tail((Nat.2 * j + Nat.1).suc) = cos_two_tail(Nat.2 * j + Nat.2)
            cos_two_tail(Nat.2 * j + Nat.1) + cos_two_tail((Nat.2 * j + Nat.1).suc) <= Real.0
            lte_add_right(cos_two_tail(Nat.2 * j + Nat.1) + cos_two_tail((Nat.2 * j + Nat.1).suc), Real.0, partial(cos_two_tail, Nat.2 * j + Nat.1))
            (cos_two_tail(Nat.2 * j + Nat.1) + cos_two_tail((Nat.2 * j + Nat.1).suc)) + partial(cos_two_tail, Nat.2 * j + Nat.1) <= Real.0 + partial(cos_two_tail, Nat.2 * j + Nat.1)
            partial(cos_two_tail, Nat.2 * j + Nat.1) + (cos_two_tail(Nat.2 * j + Nat.1) + cos_two_tail((Nat.2 * j + Nat.1).suc)) <= partial(cos_two_tail, Nat.2 * j + Nat.1)
            partial(cos_two_tail, (Nat.2 * j + Nat.1).suc.suc) <= partial(cos_two_tail, Nat.2 * j + Nat.1)
            Nat.2 * j.suc + Nat.1 = (Nat.2 * j + Nat.1).suc.suc
            partial(cos_two_tail, Nat.2 * j.suc + Nat.1) <= partial(cos_two_tail, Nat.2 * j + Nat.1)
            p(j)
            partial(cos_two_tail, Nat.2 * j + Nat.1) <= partial(cos_two_tail, Nat.1)
            lte_trans(partial(cos_two_tail, Nat.2 * j.suc + Nat.1), partial(cos_two_tail, Nat.2 * j + Nat.1), partial(cos_two_tail, Nat.1))
            partial(cos_two_tail, Nat.2 * j.suc + Nat.1) <= partial(cos_two_tail, Nat.1)
            p(j.suc)
        }
    }
    p(Nat.0) and forall(j: Nat) { p(j) implies p(j.suc) }
    Nat.induction(p)
    p(k)
}

/// The limit of the tail series at two is at most its first partial sum.
theorem cos_two_tail_limit_le_one {
    limit(partial(cos_two_tail)) <= partial(cos_two_tail, Nat.1)
} by {
    // Q(k) = P_{2k+1} is a subsequence of the tail partial sums, bounded by P_1.
    cos_two_tail_odd_subseq_converges
    converges_to(cos_two_tail_odd_partial, limit(partial(cos_two_tail)))
    converges_to_imp_converges(cos_two_tail_odd_partial, limit(partial(cos_two_tail)))
    converges(cos_two_tail_odd_partial)
    const_converges(partial(cos_two_tail, Nat.1))
    converges(const_seq(partial(cos_two_tail, Nat.1)))
    forall(k: Nat) {
        cos_two_tail_odd_le_one(k)
        partial(cos_two_tail, Nat.2 * k + Nat.1) <= partial(cos_two_tail, Nat.1)
        cos_two_tail_odd_partial(k) = partial(cos_two_tail, odd_shift_index(k))
        odd_shift_index(k) = Nat.2 * k + Nat.3
        // Q(k) = P_{2k+3} <= P_{2k+1} <= P_1
        cos_two_tail_odd_partial(k) = partial(cos_two_tail, Nat.2 * k + Nat.3)
        cos_two_tail_odd_le_one(k.suc)
        partial(cos_two_tail, Nat.2 * k.suc + Nat.1) <= partial(cos_two_tail, Nat.1)
        Nat.2 * k.suc + Nat.1 = Nat.2 * k + Nat.3
        partial(cos_two_tail, Nat.2 * k + Nat.3) <= partial(cos_two_tail, Nat.1)
        cos_two_tail_odd_partial(k) <= partial(cos_two_tail, Nat.1)
        const_seq(partial(cos_two_tail, Nat.1), k) = partial(cos_two_tail, Nat.1)
        cos_two_tail_odd_partial(k) <= const_seq(partial(cos_two_tail, Nat.1), k)
    }
    seq_lte(cos_two_tail_odd_partial, const_seq(partial(cos_two_tail, Nat.1)))
    seq_lte_preserves_limit(cos_two_tail_odd_partial, const_seq(partial(cos_two_tail, Nat.1)))
    limit(cos_two_tail_odd_partial) <= limit(const_seq(partial(cos_two_tail, Nat.1)))
    const_limit(partial(cos_two_tail, Nat.1))
    limit(const_seq(partial(cos_two_tail, Nat.1))) = partial(cos_two_tail, Nat.1)
    limit(cos_two_tail_odd_partial) <= partial(cos_two_tail, Nat.1)
    converges_imp_converges_to(cos_two_tail_odd_partial)
    converges_to(cos_two_tail_odd_partial, limit(cos_two_tail_odd_partial))
    converges_to_unique(cos_two_tail_odd_partial, limit(partial(cos_two_tail)), limit(cos_two_tail_odd_partial))
    limit(cos_two_tail_odd_partial) = limit(partial(cos_two_tail))
    limit(partial(cos_two_tail)) <= partial(cos_two_tail, Nat.1)
}

/// The cosine of two is negative.
theorem cos_two_neg {
    two.cos < Real.0
} by {
    cos_two_tail_limit_eq
    limit(partial(cos_two_tail)) = two.cos - partial(cos_term(two), Nat.2)
    cos_two_tail_limit_le_one
    limit(partial(cos_two_tail)) <= partial(cos_two_tail, Nat.1)
    two.cos - partial(cos_term(two), Nat.2) <= partial(cos_two_tail, Nat.1)
    lte_add_right(two.cos - partial(cos_term(two), Nat.2), partial(cos_two_tail, Nat.1), partial(cos_term(two), Nat.2))
    (two.cos - partial(cos_term(two), Nat.2)) + partial(cos_term(two), Nat.2) <= partial(cos_two_tail, Nat.1) + partial(cos_term(two), Nat.2)
    two.cos <= partial(cos_two_tail, Nat.1) + partial(cos_term(two), Nat.2)
    cos_two_partial_two
    partial(cos_term(two), Nat.2) = -Real.1
    two.cos <= partial(cos_two_tail, Nat.1) + -Real.1
    cos_two_tail_partial_one
    partial(cos_two_tail, Nat.1) = Real.from_rat(Rat.from_nat(Nat.2)) / Real.from_rat(Rat.from_nat(Nat.3))
    two.cos <= Real.from_rat(Rat.from_nat(Nat.2)) / Real.from_rat(Rat.from_nat(Nat.3)) + -Real.1
    // 2/3 - 1 < 0, since 2/3 < 1.
    lt_suc(Nat.2)
    Nat.2 < Nat.3
    nat_lt_imp_rat_lt(Nat.2, Nat.3)
    Rat.from_nat(Nat.2) < Rat.from_nat(Nat.3)
    Rat.from_nat(Nat.2) = Rat.2
    Rat.from_nat(Nat.3) = Rat.3
    Rat.2 < Rat.3
    from_rat_maintains_lt(Rat.2, Rat.3)
    Real.from_rat(Rat.2) < Real.from_rat(Rat.3)
    Real.from_rat(Rat.2) = Real.from_rat(Rat.from_nat(Nat.2))
    Real.from_rat(Rat.3) = Real.from_rat(Rat.from_nat(Nat.3))
    Real.from_rat(Rat.from_nat(Nat.2)) < Real.from_rat(Rat.from_nat(Nat.3))
    Real.from_rat(Rat.from_nat(Nat.3)) > Real.0
    div_lt_div_pos(Real.from_rat(Rat.from_nat(Nat.2)), Real.from_rat(Rat.from_nat(Nat.3)), Real.from_rat(Rat.from_nat(Nat.3)))
    Real.from_rat(Rat.from_nat(Nat.2)) / Real.from_rat(Rat.from_nat(Nat.3)) < Real.from_rat(Rat.from_nat(Nat.3)) / Real.from_rat(Rat.from_nat(Nat.3))
    Real.from_rat(Rat.from_nat(Nat.3)) / Real.from_rat(Rat.from_nat(Nat.3)) = Real.1
    Real.from_rat(Rat.from_nat(Nat.2)) / Real.from_rat(Rat.from_nat(Nat.3)) < Real.1
    lt_add_right(Real.from_rat(Rat.from_nat(Nat.2)) / Real.from_rat(Rat.from_nat(Nat.3)), Real.1, -Real.1)
    Real.from_rat(Rat.from_nat(Nat.2)) / Real.from_rat(Rat.from_nat(Nat.3)) + -Real.1 < Real.1 + -Real.1
    Real.1 + -Real.1 = Real.0
    Real.from_rat(Rat.from_nat(Nat.2)) / Real.from_rat(Rat.from_nat(Nat.3)) + -Real.1 < Real.0
    lte_lt_trans(two.cos,
        Real.from_rat(Rat.from_nat(Nat.2)) / Real.from_rat(Rat.from_nat(Nat.3)) + -Real.1, Real.0)
    two.cos < Real.0
}

// Section 3: The definition of pi as twice the first positive zero of cosine.

/// Membership in the set of positive zeros of cosine in (0, 2].
define cos_zero_contains(x: Real) -> Bool {
    Real.0 < x and x <= two and x.cos = Real.0
}

/// The set of positive zeros of cosine in (0, 2].
let cos_zero_set = Set[Real].new(cos_zero_contains)

/// Membership in the zero set is the defining conjunction.
theorem cos_zero_set_contains_eq(x: Real) {
    cos_zero_set.contains(x) = cos_zero_contains(x)
} by {
}

/// The zero set of cosine in (0, 2] is nonempty.
theorem cos_zero_set_nonempty {
    is_nonempty(cos_zero_set)
} by {
    cos_neg_fn_continuous
    continuous(cos_neg_fn)
    two > Real.0
    lt_imp_lte(Real.0, two)
    Real.0 <= two
    cos_neg_fn(Real.0) = -(Real.0).cos
    cos_zero
    (Real.0).cos = Real.1
    cos_neg_fn(Real.0) = -Real.1
    Real.0 < Real.1
    lt_add_right(Real.0, Real.1, -Real.1)
    Real.0 + -Real.1 < Real.1 + -Real.1
    Real.0 + -Real.1 = -Real.1
    Real.1 + -Real.1 = Real.0
    -Real.1 < Real.0
    lt_imp_lte(-Real.1, Real.0)
    -Real.1 <= Real.0
    cos_neg_fn(Real.0) <= Real.0
    cos_neg_fn(two) = -two.cos
    cos_two_neg
    two.cos < Real.0
    -two.cos > Real.0
    lt_imp_lte(Real.0, -two.cos)
    Real.0 <= -two.cos
    Real.0 <= cos_neg_fn(two)
    intermediate_value_closed_interval(cos_neg_fn, Real.0, two, Real.0)
    exists(point: Real) {
        closed_interval_set(Real.0, two).contains(point) and cos_neg_fn(point) = Real.0
    }
    let point: Real satisfy {
        closed_interval_set(Real.0, two).contains(point) and cos_neg_fn(point) = Real.0
    }
    closed_interval_set(Real.0, two).contains(point) and cos_neg_fn(point) = Real.0
    cos_neg_fn(point) = Real.0
    cos_neg_fn(point) = -point.cos
    -point.cos = Real.0
    point.cos = Real.0
    closed_interval_set(Real.0, two).contains(point)
    closed_interval_set_contains_eq(Real.0, two, point)
    closed_interval_set(Real.0, two).contains(point) = closed_interval(Real.0, two, point)
    closed_interval(Real.0, two, point)
    closed_interval(Real.0, two, point) = (Real.0 <= point and point <= two)
    Real.0 <= point and point <= two
    Real.0 <= point
    point <= two
    if point = Real.0 {
        point.cos = (Real.0).cos
        cos_zero
        (Real.0).cos = Real.1
        point.cos = Real.1
        point.cos = Real.0
        false
    }
    point != Real.0
    Real.0 <= point
    not_lt_imp_gte[Real](point, Real.0)
    not_gte_imp_lt[Real](Real.0, point)
    Real.0 < point
    cos_zero_contains(point)
    cos_zero_set_contains_eq(point)
    cos_zero_set.contains(point)
    exists(x: Real) {
        cos_zero_set.contains(x)
    }
    is_nonempty(cos_zero_set)
}

/// Zero is a lower bound of the zero set.
theorem cos_zero_set_lower_bound_zero {
    is_set_lower_bound(cos_zero_set, Real.0)
} by {
    forall(x: Real) {
        if cos_zero_set.contains(x) {
            cos_zero_set_contains_eq(x)
            cos_zero_contains(x)
            cos_zero_contains(x) = (Real.0 < x and x <= two and x.cos = Real.0)
            Real.0 < x
            lt_imp_lte(Real.0, x)
            Real.0 <= x
        }
    }
    is_set_lower_bound(cos_zero_set, Real.0)
}

/// Two is an upper bound of the zero set.
theorem cos_zero_set_upper_bound_two {
    is_set_upper_bound(cos_zero_set, two)
} by {
    forall(x: Real) {
        if cos_zero_set.contains(x) {
            cos_zero_set_contains_eq(x)
            cos_zero_contains(x)
            cos_zero_contains(x) = (Real.0 < x and x <= two and x.cos = Real.0)
            x <= two
        }
    }
    is_set_upper_bound(cos_zero_set, two)
}

/// The negation of the zero set is nonempty.
theorem cos_zero_set_neg_nonempty {
    is_nonempty(negate_set(cos_zero_set))
} by {
    cos_zero_set_nonempty
    is_nonempty(cos_zero_set)
    exists(x: Real) {
        cos_zero_set.contains(x)
    }
    let x: Real satisfy {
        cos_zero_set.contains(x)
    }
    cos_zero_set.contains(x)
    negate_set_contains(cos_zero_set, -x)
    exists(a: Real) {
        cos_zero_set.contains(a) and -x = -a
    }
    cos_zero_set.contains(x) and -x = -x
    negate_set(cos_zero_set).contains(-x)
    exists(y: Real) {
        negate_set(cos_zero_set).contains(y)
    }
    is_nonempty(negate_set(cos_zero_set))
}

/// Zero is an upper bound of the negation of the zero set.
theorem cos_zero_set_neg_upper_bound_zero {
    is_set_upper_bound(negate_set(cos_zero_set), Real.0)
} by {
    forall(y: Real) {
        if negate_set(cos_zero_set).contains(y) {
            negate_set_contains(cos_zero_set, y)
            exists(a: Real) {
                cos_zero_set.contains(a) and y = -a
            }
            let a: Real satisfy {
                cos_zero_set.contains(a) and y = -a
            }
            cos_zero_set.contains(a) and y = -a
            y = -a
            cos_zero_set.contains(a)
            cos_zero_set_contains_eq(a)
            cos_zero_contains(a)
            cos_zero_contains(a) = (Real.0 < a and a <= two and a.cos = Real.0)
            Real.0 < a
            -a < Real.0
            y < Real.0
            lt_imp_lte(y, Real.0)
            y <= Real.0
        }
    }
    is_set_upper_bound(negate_set(cos_zero_set), Real.0)
}

/// The negation of the zero set is bounded above.
theorem cos_zero_set_neg_has_upper_bound {
    has_upper_bound(negate_set(cos_zero_set))
} by {
    cos_zero_set_neg_upper_bound_zero
    is_set_upper_bound(negate_set(cos_zero_set), Real.0)
    exists(b: Real) {
        is_set_upper_bound(negate_set(cos_zero_set), b)
    }
    has_upper_bound(negate_set(cos_zero_set))
}


/// The zero set of cosine has an infimum.
/// Negating a set twice recovers the set.
theorem negate_negate_set_eq(s: Set[Real]) {
    negate_set(negate_set(s)) = s
} by {
    forall(x: Real) {
        if negate_set(negate_set(s)).contains(x) {
            negate_set_contains(negate_set(s), x)
            exists(a: Real) {
                negate_set(s).contains(a) and x = -a
            }
            let a: Real satisfy {
                negate_set(s).contains(a) and x = -a
            }
            negate_set(s).contains(a) and x = -a
            negate_set_contains(s, a)
            exists(b: Real) {
                s.contains(b) and a = -b
            }
            let b: Real satisfy {
                s.contains(b) and a = -b
            }
            s.contains(b) and a = -b
            s.contains(b)
            x = -a
            a = -b
            x = -(-b)
            neg_neg(b)
            -(-b) = b
            x = b
            s.contains(x)
        }
        if s.contains(x) {
            negate_set_contains(s, -x)
            exists(a: Real) {
                s.contains(a) and -x = -a
            }
            s.contains(x) and -x = -x
            negate_set(s).contains(-x)
            negate_set_contains(negate_set(s), x)
            exists(a: Real) {
                negate_set(s).contains(a) and x = -a
            }
            negate_set(s).contains(-x) and x = -(-x)
            neg_neg(x)
            -(-x) = x
            x = -(-x)
            negate_set(negate_set(s)).contains(x)
        }
        negate_set(negate_set(s)).contains(x) implies s.contains(x)
        s.contains(x) implies negate_set(negate_set(s)).contains(x)
        negate_set(negate_set(s)).contains(x) = s.contains(x)
    }
    set_ext(negate_set(negate_set(s)), s)
    negate_set(negate_set(s)) = s
}

/// The zero set of cosine has an infimum.
theorem cos_zero_set_has_infimum {
    exists(c: Real) {
        is_set_infimum(cos_zero_set, c)
    }
} by {
    cos_zero_set_neg_nonempty
    is_nonempty(negate_set(cos_zero_set))
    cos_zero_set_neg_has_upper_bound
    has_upper_bound(negate_set(cos_zero_set))
    completeness(negate_set(cos_zero_set))
    exists(sup: Real) {
        is_set_supremum(negate_set(cos_zero_set), sup)
    }
    let sup: Real satisfy {
        is_set_supremum(negate_set(cos_zero_set), sup)
    }
    is_set_supremum(negate_set(cos_zero_set), sup)
    supremum_infimum_duality(negate_set(cos_zero_set), sup)
    is_set_infimum(negate_set(negate_set(cos_zero_set)), -sup)
    negate_negate_set_eq(cos_zero_set)
    negate_set(negate_set(cos_zero_set)) = cos_zero_set
    is_set_infimum(cos_zero_set, -sup)
    exists(c: Real) {
        is_set_infimum(cos_zero_set, c)
    }
}

/// Half of pi: the infimum of the positive zeros of cosine in (0, 2].
let pi_over_two: Real satisfy {
    is_set_infimum(cos_zero_set, pi_over_two)
}

/// An infimum of a set is a lower bound of it.
theorem set_infimum_is_lower_bound(s: Set[Real], inf: Real) {
    is_set_infimum(s, inf) implies is_set_lower_bound(s, inf)
} by {
}

/// An infimum is at most every lower bound of its set.
theorem set_infimum_le_lower_bound(s: Set[Real], inf: Real, b: Real) {
    is_set_infimum(s, inf) and is_set_lower_bound(s, b) implies b <= inf
} by {
    if is_set_infimum(s, inf) and is_set_lower_bound(s, b) {
        is_set_infimum(s, inf) implies forall(b0: Real) {
            is_set_lower_bound(s, b0) implies b0 <= inf
        }
        is_set_infimum(s, inf)
        forall(b0: Real) {
            is_set_lower_bound(s, b0) implies b0 <= inf
        }
        is_set_lower_bound(s, b)
        b <= inf
    }
}

/// A member of a set is above any lower bound of that set.
theorem set_lower_bound_contains_le(s: Set[Real], bound: Real, x: Real) {
    is_set_lower_bound(s, bound) and s.contains(x) implies bound <= x
} by {
    if is_set_lower_bound(s, bound) and s.contains(x) {
        is_set_lower_bound(s, bound) implies forall(y: Real) {
            s.contains(y) implies bound <= y
        }
        is_set_lower_bound(s, bound)
        forall(y: Real) {
            s.contains(y) implies bound <= y
        }
        s.contains(x)
        bound <= x
    }
}

/// The defining property of pi over two.
theorem pi_over_two_is_infimum {
    is_set_infimum(cos_zero_set, pi_over_two)
} by {
}

/// The infimum of the zero set is a lower bound of it.
theorem pi_over_two_is_lower_bound {
    is_set_lower_bound(cos_zero_set, pi_over_two)
} by {
    pi_over_two_is_infimum
    set_infimum_is_lower_bound(cos_zero_set, pi_over_two)
    is_set_lower_bound(cos_zero_set, pi_over_two)
}

/// Half of pi is nonnegative.
theorem pi_over_two_nonneg {
    Real.0 <= pi_over_two
} by {
    pi_over_two_is_infimum
    cos_zero_set_lower_bound_zero
    is_set_lower_bound(cos_zero_set, Real.0)
    set_infimum_le_lower_bound(cos_zero_set, pi_over_two, Real.0)
    Real.0 <= pi_over_two
}

/// Half of pi is at most two.
theorem pi_over_two_le_two {
    pi_over_two <= two
} by {
    cos_zero_set_nonempty
    is_nonempty(cos_zero_set)
    exists(x: Real) {
        cos_zero_set.contains(x)
    }
    let x: Real satisfy {
        cos_zero_set.contains(x)
    }
    cos_zero_set.contains(x)
    pi_over_two_is_lower_bound
    is_set_lower_bound(cos_zero_set, pi_over_two)
    is_set_lower_bound(cos_zero_set, pi_over_two) = forall(y: Real) {
        cos_zero_set.contains(y) implies pi_over_two <= y
    }
    pi_over_two <= x
    cos_zero_set_contains_eq(x)
    cos_zero_contains(x)
    cos_zero_contains(x) = (Real.0 < x and x <= two and x.cos = Real.0)
    x <= two
    lte_trans(pi_over_two, x, two)
    pi_over_two <= two
}

/// A bound above the infimum is not a lower bound.
theorem set_bound_above_infimum_not_lower(s: Set[Real], inf: Real, bound: Real) {
    is_set_infimum(s, inf) and inf < bound implies not is_set_lower_bound(s, bound)
} by {
    if is_set_infimum(s, inf) and inf < bound {
        if is_set_lower_bound(s, bound) {
            set_infimum_le_lower_bound(s, inf, bound)
            bound <= inf
            inf < bound
            false
        }
    }
}

/// A number that is not a lower bound is exceeded by a member of the set.
theorem set_not_lower_bound_witness(s: Set[Real], bound: Real) {
    not is_set_lower_bound(s, bound) implies exists(x: Real) {
        s.contains(x) and not bound <= x
    }
} by {
    if not is_set_lower_bound(s, bound) {
        is_set_lower_bound(s, bound) = forall(x: Real) {
            s.contains(x) implies bound <= x
        }
        exists(x: Real) {
            s.contains(x) and not bound <= x
        }
    }
}

/// The infimum of a set is approached from above by points of the set.
theorem set_infimum_close_from_above(s: Set[Real], inf: Real, eps: Real) {
    is_set_infimum(s, inf) and eps.is_positive implies exists(x: Real) {
        s.contains(x) and inf <= x and x < inf + eps and x.is_close(inf, eps)
    }
} by {
    if is_set_infimum(s, inf) and eps.is_positive {
        let bound = inf + eps
        lt_add_pos(inf, eps)
        inf < inf + eps
        inf < bound
        set_bound_above_infimum_not_lower(s, inf, bound)
        set_not_lower_bound_witness(s, bound)
        let x: Real satisfy {
            s.contains(x) and not bound <= x
        }
        s.contains(x)
        not_lte_imp_gt[Real](bound, x)
        x < bound
        bound = inf + eps
        x < inf + eps
        set_infimum_is_lower_bound(s, inf)
        is_set_lower_bound(s, inf)
        set_lower_bound_contains_le(s, inf, x)
        inf <= x
        x < inf + eps
        inf - eps < inf
        lt_imp_lte(inf - eps, inf)
        inf - eps <= inf
        lte_trans(inf - eps, inf, x)
        inf - eps <= x
        lt_of_lte_of_lt(inf - eps, inf, x)
        inf - eps < x
        bounds_imp_close(x, inf, eps)
        x.is_close(inf, eps)
        exists(x2: Real) {
            s.contains(x2) and inf <= x2 and x2 < inf + eps and x2.is_close(inf, eps)
        }
    }
}

/// The cosine of pi over two is zero.
theorem cos_pi_over_two_zero {
    pi_over_two.cos = Real.0
} by {
    if pi_over_two.cos != Real.0 {
        // |c.cos| > 0
        pi_over_two.cos.abs != Real.0
        abs_gte_zero(pi_over_two)
        not pi_over_two.cos.abs.is_negative
        Real.0 < pi_over_two.cos.abs
        // eps = |c.cos| / 2 > 0
        two > Real.0
        div_lt_div_pos(Real.0, pi_over_two.cos.abs, two)
        Real.0 / two < pi_over_two.cos.abs / two
        Real.0 / two = Real.0
        Real.0 < pi_over_two.cos.abs / two
        (pi_over_two.cos.abs / two).is_positive
        cos_continuous_at(pi_over_two)
        continuous_at(Real.cos, pi_over_two)
        continuous_at(Real.cos, pi_over_two) = forall(eps: Real) {
            eps.is_positive implies exists(delta: Real) {
                delta.is_positive and continuous_condition(Real.cos, pi_over_two, delta, eps)
            }
        }
        let delta: Real satisfy {
            delta.is_positive and continuous_condition(Real.cos, pi_over_two, delta, pi_over_two.cos.abs / two)
        }
        delta.is_positive
        pi_over_two_is_infimum
        set_infimum_close_from_above(cos_zero_set, pi_over_two, delta)
        let z: Real satisfy {
            cos_zero_set.contains(z) and pi_over_two <= z and z < pi_over_two + delta and z.is_close(pi_over_two, delta)
        }
        cos_zero_set.contains(z) and pi_over_two <= z and z < pi_over_two + delta and z.is_close(pi_over_two, delta)
        z.is_close(pi_over_two, delta)
        continuous_condition(Real.cos, pi_over_two, delta, pi_over_two.cos.abs / two) = forall(y: Real) {
            y.is_close(pi_over_two, delta) implies y.cos.is_close(pi_over_two.cos, pi_over_two.cos.abs / two)
        }
        z.cos.is_close(pi_over_two.cos, pi_over_two.cos.abs / two)
        (z.cos - pi_over_two.cos).abs < pi_over_two.cos.abs / two
        cos_zero_set_contains_eq(z)
        cos_zero_contains(z)
        cos_zero_contains(z) = (Real.0 < z and z <= two and z.cos = Real.0)
        z.cos = Real.0
        z.cos - pi_over_two.cos = Real.0 - pi_over_two.cos
        Real.0 - pi_over_two.cos = -pi_over_two.cos
        (z.cos - pi_over_two.cos).abs = pi_over_two.cos.abs
        pi_over_two.cos.abs < pi_over_two.cos.abs / two
        two > Real.0
        lt_mul_pos_right(pi_over_two.cos.abs, pi_over_two.cos.abs / two, two)
        pi_over_two.cos.abs * two < (pi_over_two.cos.abs / two) * two
        (pi_over_two.cos.abs / two) * two = pi_over_two.cos.abs
        pi_over_two.cos.abs * two < pi_over_two.cos.abs
        pi_over_two.cos.abs > Real.0
        lt_mul_pos_left(Real.1, two, pi_over_two.cos.abs)
        Real.1 * pi_over_two.cos.abs < two * pi_over_two.cos.abs
        Real.1 * pi_over_two.cos.abs = pi_over_two.cos.abs
        two * pi_over_two.cos.abs = pi_over_two.cos.abs * two
        pi_over_two.cos.abs < pi_over_two.cos.abs * two
        false
    }
    not pi_over_two.cos != Real.0
    pi_over_two.cos = Real.0
}

// Section 4: Basic values of sine and cosine at multiples of pi.

/// The mathematical constant pi is twice the first positive zero of cosine.
let pi = two * pi_over_two

/// Pi is positive.
theorem pi_pos {
    pi > Real.0
} by {
    pi_over_two_nonneg
    Real.0 <= pi_over_two
    if pi_over_two = Real.0 {
        pi_over_two.cos = (Real.0).cos
        cos_pi_over_two_zero
        pi_over_two.cos = Real.0
        cos_zero
        (Real.0).cos = Real.1
        Real.0 = Real.1
        zero_is_different_than_one
        Real.0 != Real.1
        false
    }
    pi_over_two != Real.0
    if not Real.0 < pi_over_two {
        not_lt_imp_gte[Real](Real.0, pi_over_two)
        Real.0 >= pi_over_two
        pi_over_two <= Real.0
        lte_antisymm[Real](Real.0, pi_over_two)
        Real.0 = pi_over_two
        pi_over_two = Real.0
        false
    }
    Real.0 < pi_over_two
    two > Real.0
    two.is_positive
    lt_mul_pos_left(Real.0, pi_over_two, two)
    two * Real.0 < two * pi_over_two
    two * Real.0 = Real.0
    Real.0 < two * pi_over_two
    pi = two * pi_over_two
    Real.0 < pi
    pi > Real.0
}

/// Half of pi is strictly less than two.
theorem pi_over_two_lt_two {
    pi_over_two < two
} by {
    pi_over_two_le_two
    pi_over_two <= two
    if pi_over_two = two {
        pi_over_two.cos = two.cos
        cos_pi_over_two_zero
        pi_over_two.cos = Real.0
        two.cos = Real.0
        cos_two_neg
        two.cos < Real.0
        Real.0 < Real.0
        false
    }
    pi_over_two != two
    if not pi_over_two < two {
        not_lt_imp_gte[Real](pi_over_two, two)
        pi_over_two >= two
        two <= pi_over_two
        lte_antisymm[Real](pi_over_two, two)
        pi_over_two = two
        false
    }
    pi_over_two < two
}

/// Pi is less than four.
theorem pi_lt_four {
    pi < two * two
} by {
    pi_over_two_lt_two
    pi_over_two < two
    two > Real.0
    two.is_positive
    lt_mul_pos_left(pi_over_two, two, two)
    two * pi_over_two < two * two
    two * pi_over_two = pi
    pi < two * two
}

/// The double of pi over two is pi.
theorem pi_eq_double_pi_over_two {
    pi = pi_over_two + pi_over_two
} by {
    pi = two * pi_over_two
    two = Real.1 + Real.1
    two * pi_over_two = (Real.1 + Real.1) * pi_over_two
    (Real.1 + Real.1) * pi_over_two = pi_over_two + pi_over_two
    two * pi_over_two = pi_over_two + pi_over_two
    pi = pi_over_two + pi_over_two
}

/// The cosine of pi is negative one.
theorem cos_pi_neg_one {
    pi.cos = -Real.1
} by {
    pi_eq_double_pi_over_two
    pi = pi_over_two + pi_over_two
    cos_add(pi_over_two, pi_over_two)
    (pi_over_two + pi_over_two).cos = pi_over_two.cos * pi_over_two.cos - pi_over_two.sin * pi_over_two.sin
    pi.cos = pi_over_two.cos * pi_over_two.cos - pi_over_two.sin * pi_over_two.sin
    cos_pi_over_two_zero
    pi_over_two.cos = Real.0
    pi_over_two.cos * pi_over_two.cos = Real.0
    sin_sq_add_cos_sq(pi_over_two)
    pi_over_two.sin.pow(Nat.2) + pi_over_two.cos.pow(Nat.2) = Real.1
    pow_suc(pi_over_two.cos, Nat.1)
    pi_over_two.cos.pow(Nat.2) = pi_over_two.cos * pi_over_two.cos
    pi_over_two.cos.pow(Nat.2) = Real.0
    pi_over_two.sin.pow(Nat.2) = Real.1
    pow_suc(pi_over_two.sin, Nat.1)
    pi_over_two.sin.pow(Nat.2) = pi_over_two.sin * pi_over_two.sin
    pi_over_two.sin * pi_over_two.sin = Real.1
    pi.cos = Real.0 - Real.1
    Real.0 - Real.1 = -Real.1
    pi.cos = -Real.1
}

/// The sine of pi is zero.
theorem sin_pi_zero {
    pi.sin = Real.0
} by {
    pi_eq_double_pi_over_two
    pi = pi_over_two + pi_over_two
    sin_add(pi_over_two, pi_over_two)
    (pi_over_two + pi_over_two).sin = pi_over_two.sin * pi_over_two.cos + pi_over_two.cos * pi_over_two.sin
    pi.sin = pi_over_two.sin * pi_over_two.cos + pi_over_two.cos * pi_over_two.sin
    cos_pi_over_two_zero
    pi_over_two.cos = Real.0
    pi_over_two.sin * pi_over_two.cos = Real.0
    pi_over_two.cos * pi_over_two.sin = Real.0
    pi.sin = Real.0 + Real.0
    Real.0 + Real.0 = Real.0
    pi.sin = Real.0
}

/// The sine of two pi is zero.
theorem sin_two_pi_zero {
    (pi + pi).sin = Real.0
} by {
    sin_add(pi, pi)
    (pi + pi).sin = pi.sin * pi.cos + pi.cos * pi.sin
    sin_pi_zero
    pi.sin = Real.0
    pi.sin * pi.cos = Real.0
    pi.cos * pi.sin = Real.0
    (pi + pi).sin = Real.0
}

/// The cosine of two pi is one.
theorem cos_two_pi_one {
    (pi + pi).cos = Real.1
} by {
    cos_add(pi, pi)
    (pi + pi).cos = pi.cos * pi.cos - pi.sin * pi.sin
    cos_pi_neg_one
    pi.cos = -Real.1
    pi.cos * pi.cos = (-Real.1) * (-Real.1)
    (-Real.1) * (-Real.1) = Real.1
    pi.cos * pi.cos = Real.1
    sin_pi_zero
    pi.sin = Real.0
    pi.sin * pi.sin = Real.0
    (pi + pi).cos = Real.1 - Real.0
    Real.1 - Real.0 = Real.1
    (pi + pi).cos = Real.1
}

// Section 5: The sine of pi over two is one.

/// Half of pi is positive.
theorem pi_over_two_pos {
    Real.0 < pi_over_two
} by {
    pi_over_two_nonneg
    Real.0 <= pi_over_two
    if pi_over_two = Real.0 {
        pi_over_two.cos = (Real.0).cos
        cos_pi_over_two_zero
        pi_over_two.cos = Real.0
        cos_zero
        (Real.0).cos = Real.1
        Real.0 = Real.1
        zero_is_different_than_one
        Real.0 != Real.1
        false
    }
    pi_over_two != Real.0
    if not Real.0 < pi_over_two {
        not_lt_imp_gte[Real](Real.0, pi_over_two)
        Real.0 >= pi_over_two
        pi_over_two <= Real.0
        lte_antisymm[Real](Real.0, pi_over_two)
        Real.0 = pi_over_two
        pi_over_two = Real.0
        false
    }
    Real.0 < pi_over_two
}

/// Cosine is nonnegative on the interval from zero to pi over two.
theorem cos_nonneg_on_zero_to_pi_over_two(x: Real) {
    Real.0 <= x and x <= pi_over_two implies Real.0 <= x.cos
} by {
    if Real.0 <= x and x <= pi_over_two {
        if x.cos < Real.0 {
            if x = Real.0 {
                x.cos = (Real.0).cos
                cos_zero
                (Real.0).cos = Real.1
                x.cos = Real.1
                x.cos < Real.0
                Real.1 < Real.0
                Real.0 < Real.1
                lt_imp_lte(Real.1, Real.0)
                Real.1 <= Real.0
                lt_imp_lte(Real.0, Real.1)
                Real.0 <= Real.1
                lte_antisymm[Real](Real.1, Real.0)
                Real.1 = Real.0
                zero_is_different_than_one
                Real.0 != Real.1
                false
            }
            x != Real.0
            Real.0 <= x
            if not Real.0 < x {
                not_lt_imp_gte[Real](Real.0, x)
                Real.0 >= x
                x <= Real.0
                lte_antisymm[Real](Real.0, x)
                Real.0 = x
                x = Real.0
                false
            }
            Real.0 < x
            cos_neg_fn_continuous
            continuous(cos_neg_fn)
            Real.0 <= x
            cos_neg_fn(Real.0) = -(Real.0).cos
            cos_zero
            (Real.0).cos = Real.1
            cos_neg_fn(Real.0) = -Real.1
            Real.0 < Real.1
            lt_add_right(Real.0, Real.1, -Real.1)
            Real.0 + -Real.1 < Real.1 + -Real.1
            Real.0 + -Real.1 = -Real.1
            Real.1 + -Real.1 = Real.0
            -Real.1 < Real.0
            lt_imp_lte(-Real.1, Real.0)
            -Real.1 <= Real.0
            cos_neg_fn(Real.0) <= Real.0
            cos_neg_fn(x) = -x.cos
            x.cos < Real.0
            -x.cos > Real.0
            lt_imp_lte(Real.0, -x.cos)
            Real.0 <= -x.cos
            Real.0 <= cos_neg_fn(x)
            intermediate_value_closed_interval(cos_neg_fn, Real.0, x, Real.0)
            exists(z: Real) {
                closed_interval_set(Real.0, x).contains(z) and cos_neg_fn(z) = Real.0
            }
            let z: Real satisfy {
                closed_interval_set(Real.0, x).contains(z) and cos_neg_fn(z) = Real.0
            }
            closed_interval_set(Real.0, x).contains(z) and cos_neg_fn(z) = Real.0
            cos_neg_fn(z) = Real.0
            cos_neg_fn(z) = -z.cos
            -z.cos = Real.0
            z.cos = Real.0
            closed_interval_set_contains_eq(Real.0, x, z)
            closed_interval_set(Real.0, x).contains(z) = closed_interval(Real.0, x, z)
            closed_interval(Real.0, x, z)
            closed_interval(Real.0, x, z) = (Real.0 <= z and z <= x)
            Real.0 <= z and z <= x
            Real.0 <= z
            z <= x
            if z = Real.0 {
                z.cos = (Real.0).cos
                cos_zero
                (Real.0).cos = Real.1
                z.cos = Real.1
                z.cos = Real.0
                false
            }
            z != Real.0
            Real.0 <= z
            if not Real.0 < z {
                not_lt_imp_gte[Real](Real.0, z)
                Real.0 >= z
                z <= Real.0
                lte_antisymm[Real](Real.0, z)
                Real.0 = z
                z = Real.0
                false
            }
            Real.0 < z
            x <= pi_over_two
            if x = pi_over_two {
                x.cos = pi_over_two.cos
                cos_pi_over_two_zero
                pi_over_two.cos = Real.0
                x.cos = Real.0
                x.cos < Real.0
                Real.0 < Real.0
                false
            }
            x != pi_over_two
            if not x < pi_over_two {
                not_lt_imp_gte[Real](x, pi_over_two)
                x >= pi_over_two
                pi_over_two <= x
                lte_antisymm[Real](x, pi_over_two)
                x = pi_over_two
                false
            }
            x < pi_over_two
            lte_lt_trans(z, x, pi_over_two)
            z < pi_over_two
            pi_over_two_le_two
            pi_over_two <= two
            lte_trans(x, pi_over_two, two)
            x <= two
            lte_trans(z, x, two)
            z <= two
            z.cos = Real.0
            cos_zero_contains(z)
            cos_zero_set_contains_eq(z)
            cos_zero_set.contains(z)
            pi_over_two_is_lower_bound
            is_set_lower_bound(cos_zero_set, pi_over_two)
            set_lower_bound_contains_le(cos_zero_set, pi_over_two, z)
            pi_over_two <= z
            z < pi_over_two
            false
        }
        not x.cos < Real.0
        not_lt_imp_gte[Real](x.cos, Real.0)
        x.cos >= Real.0
        Real.0 <= x.cos
    }
}

/// The sine of pi over two is nonnegative.
theorem sin_pi_over_two_nonneg {
    Real.0 <= pi_over_two.sin
} by {
    if pi_over_two.sin < Real.0 {
        sin_continuous
        continuous(Real.sin)
        pi_over_two_pos
        Real.0 < pi_over_two
        mean_value_theorem(Real.sin, Real.cos, Real.0, pi_over_two)
        exists(k: Real) {
            Real.0 < k and k < pi_over_two and has_derivative_at(Real.sin, k, secant_slope(Real.sin, Real.0, pi_over_two))
        }
        let k: Real satisfy {
            Real.0 < k and k < pi_over_two and has_derivative_at(Real.sin, k, secant_slope(Real.sin, Real.0, pi_over_two))
        }
        Real.0 < k and k < pi_over_two and has_derivative_at(Real.sin, k, secant_slope(Real.sin, Real.0, pi_over_two))
        Real.0 < k
        k < pi_over_two
        has_derivative_at(Real.sin, k, secant_slope(Real.sin, Real.0, pi_over_two))
        sin_is_derivative_fn
        is_derivative_fn_at(Real.sin, Real.cos, k)
        has_derivative_at(Real.sin, k, k.cos)
        has_derivative_at_unique(Real.sin, k, k.cos, secant_slope(Real.sin, Real.0, pi_over_two))
        k.cos = secant_slope(Real.sin, Real.0, pi_over_two)
        secant_slope(Real.sin, Real.0, pi_over_two) = (pi_over_two.sin - (Real.0).sin) / (pi_over_two - Real.0)
        sin_zero
        (Real.0).sin = Real.0
        pi_over_two.sin - (Real.0).sin = pi_over_two.sin
        pi_over_two - Real.0 = pi_over_two
        secant_slope(Real.sin, Real.0, pi_over_two) = pi_over_two.sin / pi_over_two
        pi_over_two.sin < Real.0
        Real.0 < pi_over_two
        div_lt_div_pos(pi_over_two.sin, Real.0, pi_over_two)
        pi_over_two.sin / pi_over_two < Real.0 / pi_over_two
        Real.0 / pi_over_two = Real.0
        pi_over_two.sin / pi_over_two < Real.0
        secant_slope(Real.sin, Real.0, pi_over_two) < Real.0
        k.cos < Real.0
        Real.0 < k
        lt_imp_lte(Real.0, k)
        Real.0 <= k
        k < pi_over_two
        lt_imp_lte(k, pi_over_two)
        k <= pi_over_two
        cos_nonneg_on_zero_to_pi_over_two(k)
        Real.0 <= k.cos
        false
    }
    not pi_over_two.sin < Real.0
    not_lt_imp_gte[Real](pi_over_two.sin, Real.0)
    pi_over_two.sin >= Real.0
    Real.0 <= pi_over_two.sin
}

/// The square of the sine of pi over two is one.
theorem sin_pi_over_two_sq_one {
    pi_over_two.sin.pow(Nat.2) = Real.1
} by {
    sin_sq_add_cos_sq(pi_over_two)
    pi_over_two.sin.pow(Nat.2) + pi_over_two.cos.pow(Nat.2) = Real.1
    cos_pi_over_two_zero
    pi_over_two.cos = Real.0
    pow_suc(pi_over_two.cos, Nat.1)
    pi_over_two.cos.pow(Nat.2) = pi_over_two.cos * pi_over_two.cos
    pi_over_two.cos * pi_over_two.cos = Real.0
    pi_over_two.cos.pow(Nat.2) = Real.0
    pi_over_two.sin.pow(Nat.2) = Real.1
}

/// A nonnegative real whose square is one is one.
theorem sq_eq_one_nonneg_imp_one(a: Real) {
    a.pow(Nat.2) = Real.1 and a >= Real.0 implies a = Real.1
} by {
    if a.pow(Nat.2) = Real.1 and a >= Real.0 {
        pow_suc(a, Nat.1)
        a.pow(Nat.2) = a * a
        a * a = Real.1
        a * a - Real.1 = Real.0
        mul_sub_distrib_left(a, Real.1, a + Real.1)
        (a - Real.1) * (a + Real.1) = a * (a + Real.1) - Real.1 * (a + Real.1)
        mul_distrib_right(a, a, Real.1)
        a * (a + Real.1) = a * a + a * Real.1
        a * Real.1 = a
        a * (a + Real.1) = a * a + a
        mul_distrib_right(Real.1, a, Real.1)
        Real.1 * (a + Real.1) = Real.1 * a + Real.1 * Real.1
        Real.1 * a = a
        Real.1 * Real.1 = Real.1
        Real.1 * (a + Real.1) = a + Real.1
        (a - Real.1) * (a + Real.1) = (a * a + a) - (a + Real.1)
        (a * a + a) - (a + Real.1) = a * a - Real.1
        (a - Real.1) * (a + Real.1) = a * a - Real.1
        (a - Real.1) * (a + Real.1) = Real.0
        real_no_zero_divisors(a - Real.1, a + Real.1)
        (a - Real.1) = Real.0 or (a + Real.1) = Real.0
        if a - Real.1 = Real.0 {
            a = Real.1
        } else {
            not a - Real.1 = Real.0
            (a + Real.1) = Real.0
            (a + Real.1) + -Real.1 = Real.0 + -Real.1
            a + (Real.1 + -Real.1) = Real.0 + -Real.1
            Real.1 + -Real.1 = Real.0
            a + Real.0 = Real.0 + -Real.1
            a + Real.0 = a
            Real.0 + -Real.1 = -Real.1
            a = -Real.1
            a >= Real.0
            -Real.1 >= Real.0
            Real.0 <= -Real.1
            Real.0 < Real.1
            lt_add_right(Real.0, Real.1, -Real.1)
            Real.0 + -Real.1 < Real.1 + -Real.1
            Real.0 + -Real.1 = -Real.1
            Real.1 + -Real.1 = Real.0
            -Real.1 < Real.0
            lt_imp_lte(-Real.1, Real.0)
            -Real.1 <= Real.0
            lte_antisymm[Real](Real.0, -Real.1)
            Real.0 = -Real.1
            Real.0 + Real.1 = -Real.1 + Real.1
            Real.0 + Real.1 = Real.1
            -Real.1 + Real.1 = Real.0
            Real.1 = Real.0
            zero_is_different_than_one
            Real.0 != Real.1
            false
        }
        a = Real.1
    }
}

/// The sine of pi over two is one.
theorem sin_pi_over_two_one {
    pi_over_two.sin = Real.1
} by {
    sin_pi_over_two_sq_one
    pi_over_two.sin.pow(Nat.2) = Real.1
    sin_pi_over_two_nonneg
    Real.0 <= pi_over_two.sin
    sq_eq_one_nonneg_imp_one(pi_over_two.sin)
    pi_over_two.sin = Real.1
}

// Euler's identity: e^(i*pi) + 1 = 0.
//
// The library has no complex exponential yet, so this is stated as a goal for
// future work.  Proving it requires:
//   1. a complex exponential defined by its power series z.exp = sum z^n / n!,
//      following the real-series development in real/Real.exp.ac and real/cauchy.ac
//      (nothing complex-analytic exists yet: there is no complex Real.exp, and no
//      theorem relating complex and real exponentials);
//   2. the Euler formula (i*x).exp = x.cos + i*x.sin, obtained by splitting
//      the complex series into real and imaginary parts, using the real sine
//      and cosine series from real/trig.ac;
//   3. then (i*pi).exp = pi.cos + i*pi.sin = -1 + i*0 = -1 by the values
//      proved in this file (cos_pi_neg_one, sin_pi_zero), giving e^(i*pi) + 1 = 0.
//
// With the infrastructure above in place, the statement would be:
//   theorem euler_identity {
//       complex_exp(Complex.i * Real.of_real(pi)) + Complex.one = Complex.zero
//   }

// A tighter lower bound 2 < pi is also open.  From (pi/2).sin = 1 and the mean
// value theorem on [0, pi/2], cosine takes the value 2/pi on (0, pi/2), and
// 2/pi <= 1 gives pi >= 2; the strict inequality needs x.cos < 1 on (0, pi/2),
// which requires the missing derivative of cosine (Real.cos' = -Real.sin) and the
// monotonicity machinery that builds on it.
