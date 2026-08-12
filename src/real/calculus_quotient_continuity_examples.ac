/// Global continuity consumers for the real calculus quotient API.

from real.real_base import Real
from real.continuity_base import continuous, continuous_at
from real.continuity_affine import affine_real
from real.continuity_square import square_real
from real.calculus_api import is_derivative_fn, differentiable_everywhere,
    is_derivative_fn_imp_continuous_at, differentiable_everywhere_imp_continuous_at
from real.calculus_quotient_api import nonvanishing_everywhere,
    differentiable_everywhere_reciprocal, differentiable_everywhere_quotient
from real.calculus_named_quotient_examples import differentiable_everywhere_reciprocal_affine_real,
    differentiable_everywhere_reciprocal_square_real,
    differentiable_everywhere_affine_over_affine,
    differentiable_everywhere_square_over_affine,
    differentiable_everywhere_affine_over_square
from real.derivative_quotient import pointwise_div_real, pointwise_reciprocal_real

/// A global derivative function makes its function continuous everywhere.
theorem is_derivative_fn_imp_continuous(f: Real -> Real, df: Real -> Real) {
    is_derivative_fn(f, df) implies continuous(f)
} by {
    if is_derivative_fn(f, df) {
        forall(x: Real) {
            is_derivative_fn_imp_continuous_at(f, df, x)
            continuous_at(f, x)
        }
        continuous(f)
    }
}

/// A differentiable-everywhere function is continuous everywhere.
theorem differentiable_everywhere_imp_continuous(f: Real -> Real) {
    differentiable_everywhere(f) implies continuous(f)
} by {
    if differentiable_everywhere(f) {
        forall(x: Real) {
            differentiable_everywhere_imp_continuous_at(f, x)
            continuous_at(f, x)
        }
        continuous(f)
    }
}

/// The reciprocal of a differentiable nonvanishing function is continuous everywhere.
theorem continuous_reciprocal_of_differentiable_everywhere_nonvanishing(g: Real -> Real) {
    differentiable_everywhere(g) and nonvanishing_everywhere(g)
        implies continuous(pointwise_reciprocal_real(g))
} by {
    if differentiable_everywhere(g) and nonvanishing_everywhere(g) {
        differentiable_everywhere_reciprocal(g)
        differentiable_everywhere(pointwise_reciprocal_real(g))
        differentiable_everywhere_imp_continuous(pointwise_reciprocal_real(g))
        continuous(pointwise_reciprocal_real(g))
    }
}

/// The quotient of differentiable functions is continuous everywhere under a nonvanishing denominator.
theorem continuous_quotient_of_differentiable_everywhere_nonvanishing(f: Real -> Real, g: Real -> Real) {
    differentiable_everywhere(f) and differentiable_everywhere(g) and nonvanishing_everywhere(g)
        implies continuous(pointwise_div_real(f, g))
} by {
    if differentiable_everywhere(f) and differentiable_everywhere(g) and nonvanishing_everywhere(g) {
        differentiable_everywhere_quotient(f, g)
        differentiable_everywhere(pointwise_div_real(f, g))
        differentiable_everywhere_imp_continuous(pointwise_div_real(f, g))
        continuous(pointwise_div_real(f, g))
    }
}

/// The reciprocal of a nonvanishing named affine function is continuous everywhere.
theorem continuous_reciprocal_affine_real(a: Real, b: Real) {
    nonvanishing_everywhere(affine_real(a, b)) implies
        continuous(pointwise_reciprocal_real(affine_real(a, b)))
} by {
    if nonvanishing_everywhere(affine_real(a, b)) {
        differentiable_everywhere_reciprocal_affine_real(a, b)
        differentiable_everywhere(pointwise_reciprocal_real(affine_real(a, b)))
        differentiable_everywhere_imp_continuous(pointwise_reciprocal_real(affine_real(a, b)))
        continuous(pointwise_reciprocal_real(affine_real(a, b)))
    }
}

/// The reciprocal of a nonvanishing named square function is continuous everywhere.
theorem continuous_reciprocal_square_real {
    nonvanishing_everywhere(square_real) implies
        continuous(pointwise_reciprocal_real(square_real))
} by {
    if nonvanishing_everywhere(square_real) {
        differentiable_everywhere_reciprocal_square_real
        differentiable_everywhere(pointwise_reciprocal_real(square_real))
        differentiable_everywhere_imp_continuous(pointwise_reciprocal_real(square_real))
        continuous(pointwise_reciprocal_real(square_real))
    }
}

/// A quotient of two named affine functions is continuous everywhere under a nonvanishing denominator.
theorem continuous_affine_over_affine(a: Real, b: Real, c: Real, d: Real) {
    nonvanishing_everywhere(affine_real(c, d)) implies
        continuous(pointwise_div_real(affine_real(a, b), affine_real(c, d)))
} by {
    if nonvanishing_everywhere(affine_real(c, d)) {
        differentiable_everywhere_affine_over_affine(a, b, c, d)
        differentiable_everywhere(pointwise_div_real(affine_real(a, b), affine_real(c, d)))
        differentiable_everywhere_imp_continuous(pointwise_div_real(affine_real(a, b), affine_real(c, d)))
        continuous(pointwise_div_real(affine_real(a, b), affine_real(c, d)))
    }
}

/// The quotient of the square function by a named affine denominator is continuous everywhere when the denominator is nonvanishing.
theorem continuous_square_over_affine(a: Real, b: Real) {
    nonvanishing_everywhere(affine_real(a, b)) implies
        continuous(pointwise_div_real(square_real, affine_real(a, b)))
} by {
    if nonvanishing_everywhere(affine_real(a, b)) {
        differentiable_everywhere_square_over_affine(a, b)
        differentiable_everywhere(pointwise_div_real(square_real, affine_real(a, b)))
        differentiable_everywhere_imp_continuous(pointwise_div_real(square_real, affine_real(a, b)))
        continuous(pointwise_div_real(square_real, affine_real(a, b)))
    }
}

/// The quotient of a named affine numerator by the square function is continuous everywhere when the denominator is nonvanishing.
theorem continuous_affine_over_square(a: Real, b: Real) {
    nonvanishing_everywhere(square_real) implies
        continuous(pointwise_div_real(affine_real(a, b), square_real))
} by {
    if nonvanishing_everywhere(square_real) {
        differentiable_everywhere_affine_over_square(a, b)
        differentiable_everywhere(pointwise_div_real(affine_real(a, b), square_real))
        differentiable_everywhere_imp_continuous(pointwise_div_real(affine_real(a, b), square_real))
        continuous(pointwise_div_real(affine_real(a, b), square_real))
    }
}
