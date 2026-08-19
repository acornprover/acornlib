/// Deep properties of the real logarithm.
///
/// This file extends `log.ac`, `log_inequalities.ac` and `log_exp_foundations.ac`
/// with multi-factor logarithm identities, order-theoretic characterizations,
/// sign characterizations, and classical values such as `log 2`.

from nat import Nat, from_nat
from order import lt_trans, lt_of_lte_of_lt, not_lt_imp_gte, not_lte_imp_gt, not_gte_imp_lt, not_gt_imp_lte
from real.log import Real, log_some_of_pos_exists, log_one, log_exp, exp_log_or_zero, exp_injective, exp_neg
from real.log_exp_foundations import log_value_one, log_value_e, log_value_exp, log_value_mul, log_value_div, log_value_recip, log_value_rpow, log_value_monotone, log_le_imp_le, log_ge_imp_ge, log_lt_imp_lt, log_injective, log_mul_val, log_nat_pow
from real.log_inequalities import log_le_sub_one, one_sub_recip_le_log, log_nonneg_of_ge_one, log_nonpos_of_pos_le_one, log_pos_of_gt_one, log_neg_of_pos_lt_one
from real.exp_log_properties import log_strictly_increasing, log_one_plus_x_lt_x
from real.exp import exp_pos, exp_zero, exp_increasing, exp_add, two, two_positive, three, e_gt_two, e_less_than_three, from_nat_two_eq, pow_pos
from real.exp_inequalities import exp_monotone, exp_ge_one_of_nonneg, exp_le_one_of_nonpos, exp_lt_one_of_neg
from real.real_ring import mul_nonneg, mul_pos_pos, lte_mul_nonneg_right, lte_mul_nonneg_left, mul_le_mul_nonneg, from_nat_is_from_rat, real_mul_comm
from real.real_base import lt_add_pos, lte_lt_trans, lte_add_right, lt_add_right, add_lte_add, lte_add_left, lt_lte_trans, one_half_plus_one_half, lt_swap_neg
from real.derivative_rules import div_add_distrib, neg_div
from real.harmonic import real_one_div_pos

numerals Real

// =====================================================================
// Logarithm algebraic identities
// =====================================================================

/// The logarithm of a three-factor product is the sum of the logarithms.
theorem log_value_mul3(x: Real, y: Real, z: Real) {
    x > Real.0 and y > Real.0 and z > Real.0 implies
        (x * y * z).log.get_or_else(Real.0) = x.log.get_or_else(Real.0) + y.log.get_or_else(Real.0) + z.log.get_or_else(Real.0)
} by {
    if x > Real.0 and y > Real.0 and z > Real.0 {
        x.is_positive
        y.is_positive
        (x * y).is_positive
        x * y > Real.0
        log_value_mul(x * y, z)
        ((x * y) * z).log.get_or_else(Real.0) = (x * y).log.get_or_else(Real.0) + z.log.get_or_else(Real.0)
        (x * y) * z = x * y * z
        (x * y * z).log.get_or_else(Real.0) = (x * y).log.get_or_else(Real.0) + z.log.get_or_else(Real.0)
        log_value_mul(x, y)
        (x * y).log.get_or_else(Real.0) = x.log.get_or_else(Real.0) + y.log.get_or_else(Real.0)
        (x * y * z).log.get_or_else(Real.0) = x.log.get_or_else(Real.0) + y.log.get_or_else(Real.0) + z.log.get_or_else(Real.0)
    }
}

/// The logarithm of a three-factor product with explicit logarithm values.
theorem log_mul3_val(x: Real, y: Real, z: Real, lx: Real, ly: Real, lz: Real) {
    x > Real.0 and y > Real.0 and z > Real.0 and
    x.log = Option.some(lx) and y.log = Option.some(ly) and z.log = Option.some(lz)
    implies (x * y * z).log = Option.some(lx + ly + lz)
} by {
    if x > Real.0 and y > Real.0 and z > Real.0 and
        x.log = Option.some(lx) and y.log = Option.some(ly) and z.log = Option.some(lz) {
        log_mul_val(x, y, lx, ly)
        (x * y).log = Option.some(lx + ly)
        x.is_positive
        y.is_positive
        (x * y).is_positive
        x * y > Real.0
        log_mul_val(x * y, z, lx + ly, lz)
        ((x * y) * z).log = Option.some((lx + ly) + lz)
        (x * y) * z = x * y * z
        (lx + ly) + lz = lx + ly + lz
        (x * y * z).log = Option.some(lx + ly + lz)
    }
}

/// The logarithm of a square is twice the logarithm.
theorem log_value_sq(x: Real) {
    x > Real.0 implies (x.pow(Nat.2)).log.get_or_else(Real.0) = two * x.log.get_or_else(Real.0)
} by {
    if x > Real.0 {
        log_nat_pow(x, Nat.2)
        (x.pow(Nat.2)).log = Option.some(from_nat[Real](Nat.2) * x.log.get_or_else(Real.0))
        pow_pos(x, Nat.2)
        x.pow(Nat.2) > Real.0
        log_some_of_pos_exists(x.pow(Nat.2))
        let lz: Real satisfy {
            (x.pow(Nat.2)).log = Option.some(lz)
        }
        (x.pow(Nat.2)).log.get_or_else(Real.0) = lz
        (x.pow(Nat.2)).log = Option.some((x.pow(Nat.2)).log.get_or_else(Real.0))
        Option.some((x.pow(Nat.2)).log.get_or_else(Real.0)) = Option.some(from_nat[Real](Nat.2) * x.log.get_or_else(Real.0))
        some_injective[Real]((x.pow(Nat.2)).log.get_or_else(Real.0), from_nat[Real](Nat.2) * x.log.get_or_else(Real.0))
        (x.pow(Nat.2)).log.get_or_else(Real.0) = from_nat[Real](Nat.2) * x.log.get_or_else(Real.0)
        from_nat_two_eq
        from_nat[Real](Nat.2) = two
        (x.pow(Nat.2)).log.get_or_else(Real.0) = two * x.log.get_or_else(Real.0)
    }
}

/// The logarithm of a natural power is the natural multiple of the logarithm.
theorem log_value_pow(x: Real, n: Nat) {
    x > Real.0 implies (x.pow(n)).log.get_or_else(Real.0) = from_nat[Real](n) * x.log.get_or_else(Real.0)
} by {
    if x > Real.0 {
        log_nat_pow(x, n)
        (x.pow(n)).log = Option.some(from_nat[Real](n) * x.log.get_or_else(Real.0))
        pow_pos(x, n)
        x.pow(n) > Real.0
        log_some_of_pos_exists(x.pow(n))
        let lz: Real satisfy {
            (x.pow(n)).log = Option.some(lz)
        }
        (x.pow(n)).log.get_or_else(Real.0) = lz
        (x.pow(n)).log = Option.some((x.pow(n)).log.get_or_else(Real.0))
        Option.some((x.pow(n)).log.get_or_else(Real.0)) = Option.some(from_nat[Real](n) * x.log.get_or_else(Real.0))
        some_injective[Real]((x.pow(n)).log.get_or_else(Real.0), from_nat[Real](n) * x.log.get_or_else(Real.0))
        (x.pow(n)).log.get_or_else(Real.0) = from_nat[Real](n) * x.log.get_or_else(Real.0)
    }
}

/// The logarithm of a natural power of Euler's number is the natural number.
theorem log_value_e_pow(n: Nat) {
    (Real.e.pow(n)).log.get_or_else(Real.0) = from_nat[Real](n)
} by {
    exp_pos(Real.1)
    (Real.1).exp > Real.0
    Real.e = (Real.1).exp
    Real.e > Real.0
    log_value_pow(Real.e, n)
    (Real.e.pow(n)).log.get_or_else(Real.0) = from_nat[Real](n) * (Real.e).log.get_or_else(Real.0)
    log_value_e
    (Real.e).log.get_or_else(Real.0) = Real.1
    from_nat[Real](n) * Real.1 = from_nat[Real](n)
    (Real.e.pow(n)).log.get_or_else(Real.0) = from_nat[Real](n)
}

/// The logarithm of a reciprocal exponential is the negation.
theorem log_value_exp_recip(x: Real) {
    (Real.1 / x.exp).log.get_or_else(Real.0) = -x
} by {
    exp_pos(x)
    x.exp > Real.0
    log_value_recip(x.exp)
    (Real.1 / x.exp).log.get_or_else(Real.0) = -(x.exp).log.get_or_else(Real.0)
    log_value_exp(x)
    (x.exp).log.get_or_else(Real.0) = x
    (Real.1 / x.exp).log.get_or_else(Real.0) = -x
}

/// The logarithm of a quotient by itself is zero.
theorem log_value_div_self(x: Real) {
    x > Real.0 implies (x / x).log.get_or_else(Real.0) = Real.0
} by {
    if x > Real.0 {
        x != Real.0
        x / x = Real.1
        log_value_one
        (Real.1).log.get_or_else(Real.0) = Real.0
        (x / x).log.get_or_else(Real.0) = Real.0
    }
}

// =====================================================================
// Logarithm inequalities
// =====================================================================

/// The logarithm of a positive real is at most the real minus one.
theorem log_value_le_sub_one(x: Real) {
    x > Real.0 implies x.log.get_or_else(Real.0) <= x - Real.1
} by {
    if x > Real.0 {
        log_some_of_pos_exists(x)
        let lx: Real satisfy {
            x.log = Option.some(lx)
        }
        x.log.get_or_else(Real.0) = lx
        x.log = Option.some(x.log.get_or_else(Real.0))
        log_le_sub_one(x, x.log.get_or_else(Real.0))
        x.log.get_or_else(Real.0) <= x - Real.1
    }
}

/// The logarithm of a positive real is at least one minus the reciprocal.
theorem one_sub_recip_le_log_or_zero(x: Real) {
    x > Real.0 implies Real.1 - Real.1 / x <= x.log.get_or_else(Real.0)
} by {
    if x > Real.0 {
        log_some_of_pos_exists(x)
        let lx: Real satisfy {
            x.log = Option.some(lx)
        }
        x.log.get_or_else(Real.0) = lx
        x.log = Option.some(x.log.get_or_else(Real.0))
        one_sub_recip_le_log(x, x.log.get_or_else(Real.0))
        Real.1 - Real.1 / x <= x.log.get_or_else(Real.0)
    }
}

/// The logarithm of a positive real is strictly below the real.
theorem log_value_lt_self(x: Real) {
    x > Real.0 implies x.log.get_or_else(Real.0) < x
} by {
    if x > Real.0 {
        log_value_le_sub_one(x)
        x.log.get_or_else(Real.0) <= x - Real.1
        Real.0 < Real.1
        lt_swap_neg(Real.0, Real.1)
        -Real.1 < -Real.0
        -Real.0 = Real.0
        -Real.1 < Real.0
        lt_add_right(-Real.1, Real.0, x)
        -Real.1 + x < Real.0 + x
        -Real.1 + x = x - Real.1
        Real.0 + x = x
        x - Real.1 < x
        lte_lt_trans(x.log.get_or_else(Real.0), x - Real.1, x)
        x.log.get_or_else(Real.0) < x
    }
}

/// The logarithm of one plus x is at least x over one plus x.
theorem log_one_plus_x_ge_x_over_one_plus_x(x: Real) {
    x >= Real.0 implies x / (Real.1 + x) <= (Real.1 + x).log.get_or_else(Real.0)
} by {
    if x >= Real.0 {
        Real.1 > Real.0
        Real.0 < Real.1
        lt_add_right(Real.0, Real.1, x)
        Real.0 + x < Real.1 + x
        Real.0 + x = x
        x < Real.1 + x
        x >= Real.0
        Real.0 <= x
        lt_of_lte_of_lt(Real.0, x, Real.1 + x)
        Real.0 < Real.1 + x
        Real.1 + x > Real.0
        one_sub_recip_le_log_or_zero(Real.1 + x)
        Real.1 - Real.1 / (Real.1 + x) <= (Real.1 + x).log.get_or_else(Real.0)
        Real.1 + x != Real.0
        div_add_distrib(Real.1 + x, -Real.1, Real.1 + x)
        (Real.1 + x + -Real.1) / (Real.1 + x) =
            (Real.1 + x) / (Real.1 + x) + (-Real.1) / (Real.1 + x)
        Real.1 + x + -Real.1 = x + Real.1 + -Real.1
        x + Real.1 + -Real.1 = x + (Real.1 + -Real.1)
        Real.1 + -Real.1 = Real.0
        x + (Real.1 + -Real.1) = x + Real.0
        x + Real.0 = x
        Real.1 + x + -Real.1 = x
        x / (Real.1 + x) = (Real.1 + x) / (Real.1 + x) + (-Real.1) / (Real.1 + x)
        neg_div(Real.1, Real.1 + x)
        (-Real.1) / (Real.1 + x) = -(Real.1 / (Real.1 + x))
        (Real.1 + x) / (Real.1 + x) = Real.1
        x / (Real.1 + x) = Real.1 - Real.1 / (Real.1 + x)
        x / (Real.1 + x) <= (Real.1 + x).log.get_or_else(Real.0)
    }
}

// =====================================================================
// Logarithm order characterizations
// =====================================================================

/// The logarithm preserves the strict order in both directions.
theorem log_value_lt_iff(x: Real, y: Real) {
    x > Real.0 and y > Real.0 implies (x.log.get_or_else(Real.0) < y.log.get_or_else(Real.0) iff x < y)
} by {
    if x > Real.0 and y > Real.0 {
        if x.log.get_or_else(Real.0) < y.log.get_or_else(Real.0) {
            log_lt_imp_lt(x, y)
            x < y
        }
        if x < y {
            log_strictly_increasing(x, y)
            x.log.get_or_else(Real.0) < y.log.get_or_else(Real.0)
        }
        x.log.get_or_else(Real.0) < y.log.get_or_else(Real.0) iff x < y
    }
}

/// The logarithm preserves the non-strict order in both directions.
theorem log_value_le_iff(x: Real, y: Real) {
    x > Real.0 and y > Real.0 implies (x.log.get_or_else(Real.0) <= y.log.get_or_else(Real.0) iff x <= y)
} by {
    if x > Real.0 and y > Real.0 {
        if x.log.get_or_else(Real.0) <= y.log.get_or_else(Real.0) {
            log_le_imp_le(x, y)
            x <= y
        }
        if x <= y {
            log_value_monotone(x, y)
            x.log.get_or_else(Real.0) <= y.log.get_or_else(Real.0)
        }
        x.log.get_or_else(Real.0) <= y.log.get_or_else(Real.0) iff x <= y
    }
}

/// The logarithm is injective on positive reals in both directions.
theorem log_value_eq_iff(x: Real, y: Real) {
    x > Real.0 and y > Real.0 implies (x.log.get_or_else(Real.0) = y.log.get_or_else(Real.0) iff x = y)
} by {
    if x > Real.0 and y > Real.0 {
        if x.log.get_or_else(Real.0) = y.log.get_or_else(Real.0) {
            log_injective(x, y)
            x = y
        }
        if x = y {
            x.log.get_or_else(Real.0) = y.log.get_or_else(Real.0)
        }
        x.log.get_or_else(Real.0) = y.log.get_or_else(Real.0) iff x = y
    }
}

// =====================================================================
// Logarithm sign characterizations
// =====================================================================

/// The logarithm is negative exactly between zero and one.
theorem log_value_neg_iff_lt_one(x: Real) {
    x > Real.0 implies (x.log.get_or_else(Real.0) < Real.0 iff x < Real.1)
} by {
    if x > Real.0 {
        if x.log.get_or_else(Real.0) < Real.0 {
            exp_increasing(x.log.get_or_else(Real.0), Real.0)
            (x.log.get_or_else(Real.0)).exp < (Real.0).exp
            exp_zero
            (Real.0).exp = Real.1
            log_some_of_pos_exists(x)
            let lx: Real satisfy {
                x.log = Option.some(lx)
            }
            exp_log_or_zero(x, lx)
            lx.exp = x
            x.log.get_or_else(Real.0) = lx
            (x.log.get_or_else(Real.0)).exp = x
            x < Real.1
        }
        if x < Real.1 {
            log_some_of_pos_exists(x)
            let lx: Real satisfy {
                x.log = Option.some(lx)
            }
            x.log.get_or_else(Real.0) = lx
            x.log = Option.some(x.log.get_or_else(Real.0))
            log_neg_of_pos_lt_one(x, x.log.get_or_else(Real.0))
            x.log.get_or_else(Real.0) < Real.0
        }
        x.log.get_or_else(Real.0) < Real.0 iff x < Real.1
    }
}

/// The logarithm is positive exactly above one.
theorem log_value_pos_iff_gt_one(x: Real) {
    x > Real.0 implies (x.log.get_or_else(Real.0) > Real.0 iff x > Real.1)
} by {
    if x > Real.0 {
        if x.log.get_or_else(Real.0) > Real.0 {
            exp_increasing(Real.0, x.log.get_or_else(Real.0))
            (Real.0).exp < (x.log.get_or_else(Real.0)).exp
            exp_zero
            (Real.0).exp = Real.1
            log_some_of_pos_exists(x)
            let lx: Real satisfy {
                x.log = Option.some(lx)
            }
            exp_log_or_zero(x, lx)
            lx.exp = x
            x.log.get_or_else(Real.0) = lx
            (x.log.get_or_else(Real.0)).exp = x
            Real.1 < x
            x > Real.1
        }
        if x > Real.1 {
            log_some_of_pos_exists(x)
            let lx: Real satisfy {
                x.log = Option.some(lx)
            }
            x.log.get_or_else(Real.0) = lx
            x.log = Option.some(x.log.get_or_else(Real.0))
            log_pos_of_gt_one(x, x.log.get_or_else(Real.0))
            x.log.get_or_else(Real.0) > Real.0
        }
        x.log.get_or_else(Real.0) > Real.0 iff x > Real.1
    }
}

/// The logarithm is nonnegative exactly at or above one.
theorem log_value_nonneg_iff_ge_one(x: Real) {
    x > Real.0 implies (x.log.get_or_else(Real.0) >= Real.0 iff x >= Real.1)
} by {
    if x > Real.0 {
        if x.log.get_or_else(Real.0) >= Real.0 {
            exp_monotone(Real.0, x.log.get_or_else(Real.0))
            (Real.0).exp <= (x.log.get_or_else(Real.0)).exp
            exp_zero
            (Real.0).exp = Real.1
            log_some_of_pos_exists(x)
            let lx: Real satisfy {
                x.log = Option.some(lx)
            }
            exp_log_or_zero(x, lx)
            lx.exp = x
            x.log.get_or_else(Real.0) = lx
            (x.log.get_or_else(Real.0)).exp = x
            Real.1 <= x
            x >= Real.1
        }
        if x >= Real.1 {
            log_some_of_pos_exists(x)
            let lx: Real satisfy {
                x.log = Option.some(lx)
            }
            x.log.get_or_else(Real.0) = lx
            x.log = Option.some(x.log.get_or_else(Real.0))
            log_nonneg_of_ge_one(x, x.log.get_or_else(Real.0))
            x.log.get_or_else(Real.0) >= Real.0
        }
        x.log.get_or_else(Real.0) >= Real.0 iff x >= Real.1
    }
}

/// The logarithm is nonpositive exactly between zero and one.
theorem log_value_nonpos_iff_le_one(x: Real) {
    x > Real.0 implies (x.log.get_or_else(Real.0) <= Real.0 iff x <= Real.1)
} by {
    if x > Real.0 {
        if x.log.get_or_else(Real.0) <= Real.0 {
            exp_monotone(x.log.get_or_else(Real.0), Real.0)
            (x.log.get_or_else(Real.0)).exp <= (Real.0).exp
            exp_zero
            (Real.0).exp = Real.1
            log_some_of_pos_exists(x)
            let lx: Real satisfy {
                x.log = Option.some(lx)
            }
            exp_log_or_zero(x, lx)
            lx.exp = x
            x.log.get_or_else(Real.0) = lx
            (x.log.get_or_else(Real.0)).exp = x
            x <= Real.1
        }
        if x <= Real.1 {
            log_some_of_pos_exists(x)
            let lx: Real satisfy {
                x.log = Option.some(lx)
            }
            x.log.get_or_else(Real.0) = lx
            x.log = Option.some(x.log.get_or_else(Real.0))
            log_nonpos_of_pos_le_one(x, x.log.get_or_else(Real.0))
            x.log.get_or_else(Real.0) <= Real.0
        }
        x.log.get_or_else(Real.0) <= Real.0 iff x <= Real.1
    }
}

// =====================================================================
// Classical logarithm values
// =====================================================================

/// The logarithm of two is positive.
theorem log_value_two_pos {
    two.log.get_or_else(Real.0) > Real.0
} by {
    Real.0 < Real.1
    lt_add_right(Real.0, Real.1, Real.1)
    Real.0 + Real.1 < Real.1 + Real.1
    Real.0 + Real.1 = Real.1
    Real.1 < Real.1 + Real.1
    Real.1 + Real.1 = two
    Real.1 < two
    lt_trans(Real.0, Real.1, two)
    Real.0 < two
    two > Real.0
    log_some_of_pos_exists(two)
    let lz: Real satisfy {
        two.log = Option.some(lz)
    }
    two.log.get_or_else(Real.0) = lz
    two.log = Option.some(two.log.get_or_else(Real.0))
    log_pos_of_gt_one(two, two.log.get_or_else(Real.0))
    two.log.get_or_else(Real.0) > Real.0
}

/// The logarithm of two is below one.
theorem log_value_two_lt_one {
    two.log.get_or_else(Real.0) < Real.1
} by {
    e_gt_two
    Real.e > two
    two < Real.e
    Real.1 > Real.0
    lt_trans(Real.0, Real.1, two)
    Real.0 < two
    two > Real.0
    exp_pos(Real.1)
    (Real.1).exp > Real.0
    Real.e = (Real.1).exp
    Real.e > Real.0
    log_strictly_increasing(two, Real.e)
    two.log.get_or_else(Real.0) < (Real.e).log.get_or_else(Real.0)
    log_value_e
    (Real.e).log.get_or_else(Real.0) = Real.1
    two.log.get_or_else(Real.0) < Real.1
}

/// The logarithm of the exponential of a negation is the negation.
theorem log_value_exp_neg(x: Real) {
    ((-x).exp).log.get_or_else(Real.0) = -x
} by {
    log_value_exp(-x)
    ((-x).exp).log.get_or_else(Real.0) = -x
}
