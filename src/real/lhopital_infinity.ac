/// L'Hôpital's rule for the ∞/∞ indeterminate form (one-sided, x -> a+).
///
/// This file states and proves L'Hôpital's rule for the indeterminate form
/// ∞/∞ at a point `a`, in the one-sided form x -> a+:
///
///   * `limit_at_right`: the ε-δ right-limit predicate (every tolerance is
///     met on a punctured right neighborhood (a, a + delta) contained in
///     (a, b));
///   * `tends_infinity_right`: the ε-δ "tends to +∞" predicate for x -> a+;
///   * `cauchy_mvt_ratio`: the ratio (f(b) - f(a)) / (g(b) - g(a)) equals
///     df(c)/dg(c) at the Cauchy mean value theorem point, provided dg and
///     the denominator stay nonzero;
///   * the pointwise bounds `lhopital_infinity_ratio_pointwise` and
///     `lhopital_infinity_scale_pointwise` for the two factors of the
///     identity f(x)/g(x) = R(x) * S(x) with
///     R(x) = (f(x1) - f(x)) / (g(x1) - g(x)) and
///     S(x) = (1 - g(x1)/g(x)) / (1 - f(x1)/f(x));
///   * `lhopital_infinity_core`: the ε-δ core bound, and the two final
///     theorems `lhopital_infinity_form` (the point-level rule) and
///     `lhopital_infinity_seq_limit` (the sequential form, matching the
///     style of the 0/0 rule in real/lhopital.ac).
///
/// The proof follows the classical argument: fix x1 near a, write
/// f(x)/g(x) = R(x) * S(x), bound R(x) by the derivative-ratio limit through
/// Cauchy's mean value theorem, and bound S(x) - 1 by the fact that f and g
/// tend to +∞ (so f(x1)/f(x) and g(x1)/g(x) tend to 0).

from data.basic.functions import compose
from nat import Nat
from real.lhopital import quotient_fn, cauchy_mean_value_theorem, div_not_zero
from real.mean_value import mean_value_theorem, secant_slope, lte_imp_neg_lte_neg
from real.calculus_api import is_derivative_fn, is_derivative_fn_at
from real.continuity_base import Real, continuous
from real.derivative_basic import has_derivative_at, has_derivative_at_unique, sub_ne_zero_of_ne
from real.real_seq import converges_to, tail_bound, tail_bound_implies_is_close, lt_imp_minus_pos, eps_smaller_than_both, find_less_than_a_third, close_and_lt_imp_close, sub_lt_is_gt
from real.real_base import add_comm, add_assoc, add_zero_left, add_zero_right, neg_zero, neg_neg, add_neg_eq_zero, neg_distrib, close_imp_bounds, abs_gte_zero, lte_lt_trans, lt_lte_trans, lt_add_pos, add_lt_lt, pos_imp_eq_abs, gt_zero_imp_pos, pos_gt_zero, sub_cancels, one_half_positive, one_half_plus_one_half, lt_add_right, abs_neg
from real.real_field import div_cancel_common, div_div, div_lt_mul_pos, mul_div_cancel, div_le_of_mul_le, mul_inverse, div_mul_cancel_left, mul_left_cancel, mul_div, mul_le_mul_pos_right
from real.real_ring import real_mul_comm, mul_neg_left, mul_neg_right, mul_abs, mul_pos_pos, mul_one_right, mul_one_left, mul_distrib_left, mul_distrib_right, mul_sub_distrib_right, mul_assoc
from real.real_series import triangle_ineq
from real.am_gm import div_pos_of_pos_pos, div_nonneg_of_nonneg_pos
from real.exp import abs_div, two, two_positive, two_nonzero, div_lt_div_pos
from real.convex import neg_div_neg_eq
from real.derivative_continuity import div_mul_cancel_denominator
from real.uniform_continuity import div_le_div_pos
from real.derivative_rules import div_add_distrib, neg_div
from real.trig_identities import div_sub_same_denom
from ordered_field import mul_lt_mul_of_pos_right, multiply_inequality_with_nonnegative_element, inverse_of_positive_is_positive, inverse_on_positive_flips_inequality
from algebra.add_ordered_group import add_le_add_right
from algebra.field.field import inverse_not_zero, mul_not_zero
from order import lt_trans, lt_imp_lte, lte_trans, lt_imp_ne

numerals Real

// =============================================================================
// One-sided limit predicates
// =============================================================================

/// True if f has right limit `l` at a (as x -> a+ from above), in ε-δ form on
/// the punctured right neighborhood (a, a + delta) contained in (a, b).
define limit_at_right(f: Real -> Real, a: Real, b: Real, l: Real) -> Bool {
    forall(eps: Real) {
        eps.is_positive implies exists(delta: Real) {
            delta.is_positive and a + delta < b and
            forall(x: Real) {
                a < x and x < a + delta implies f(x).is_close(l, eps)
            }
        }
    }
}

/// True if f tends to +∞ as x -> a+ from above (restricted to (a, b)).
define tends_infinity_right(f: Real -> Real, a: Real, b: Real) -> Bool {
    forall(m: Real) {
        exists(delta: Real) {
            delta.is_positive and a + delta < b and
            forall(x: Real) {
                a < x and x < a + delta implies m < f(x)
            }
        }
    }
}

/// Applying the right-limit hypothesis at a tolerance yields a delta.
theorem limit_at_right_apply(f: Real -> Real, a: Real, b: Real, l: Real, eps: Real) {
    limit_at_right(f, a, b, l) and eps.is_positive
    implies exists(delta: Real) {
        delta.is_positive and a + delta < b and
        forall(x: Real) {
            a < x and x < a + delta implies f(x).is_close(l, eps)
        }
    }
} by {
    if limit_at_right(f, a, b, l) and eps.is_positive {
        limit_at_right(f, a, b, l) = forall(eps2: Real) {
            eps2.is_positive implies exists(delta2: Real) {
                delta2.is_positive and a + delta2 < b and
                forall(x: Real) {
                    a < x and x < a + delta2 implies f(x).is_close(l, eps2)
                }
            }
        }
        exists(delta: Real) {
            delta.is_positive and a + delta < b and
            forall(x: Real) {
                a < x and x < a + delta implies f(x).is_close(l, eps)
            }
        }
    }
}

/// Applying the tends-to-infinity hypothesis at a bound yields a delta.
theorem tends_infinity_right_apply(f: Real -> Real, a: Real, b: Real, m: Real) {
    tends_infinity_right(f, a, b)
    implies exists(delta: Real) {
        delta.is_positive and a + delta < b and
        forall(x: Real) {
            a < x and x < a + delta implies m < f(x)
        }
    }
} by {
    if tends_infinity_right(f, a, b) {
        tends_infinity_right(f, a, b) = forall(m2: Real) {
            exists(delta2: Real) {
                delta2.is_positive and a + delta2 < b and
                forall(x: Real) {
                    a < x and x < a + delta2 implies m2 < f(x)
                }
            }
        }
        exists(delta: Real) {
            delta.is_positive and a + delta < b and
            forall(x: Real) {
                a < x and x < a + delta implies m < f(x)
            }
        }
    }
}

// =============================================================================
// The two factors of the identity f(x)/g(x) = R(x) * S(x)
// =============================================================================

/// The fixed-endpoint ratio (f(x1) - f(x)) / (g(x1) - g(x)) used in the ∞/∞
/// proof: by Cauchy's mean value theorem it equals df(c)/dg(c) for some c
/// between x and x1.
define lhopital_ratio_x1(f: Real -> Real, g: Real -> Real, x1: Real, x: Real) -> Real {
    (f(x1) - f(x)) / (g(x1) - g(x))
}

/// The scale factor (1 - g(x1)/g(x)) / (1 - f(x1)/f(x)) used in the ∞/∞
/// proof: it tends to 1 when f and g tend to +∞, and
/// f(x)/g(x) = lhopital_ratio_x1(f, g, x1, x) * lhopital_scale_x1(f, g, x1, x).
define lhopital_scale_x1(f: Real -> Real, g: Real -> Real, x1: Real, x: Real) -> Real {
    (Real.1 - g(x1) / g(x)) / (Real.1 - f(x1) / f(x))
}

// =============================================================================
// Small algebraic lemmas
// =============================================================================

/// If c < e * d and d is positive, then c / d < e.
theorem lt_mul_pos_imp_div_lt(c: Real, d: Real, e: Real) {
    c < e * d and d > Real.0 implies c / d < e
} by {
    if c < e * d and d > Real.0 {
        lt_imp_ne(Real.0, d)
        Real.0 != d
        d != Real.0
        inverse_of_positive_is_positive[Real](d)
        d.inverse > Real.0
        mul_lt_mul_of_pos_right[Real](c, e * d, d.inverse)
        c * d.inverse < (e * d) * d.inverse
        (e * d) * d.inverse = e * (d * d.inverse)
        mul_inverse(d)
        d * d.inverse = Real.1
        e * (d * d.inverse) = e * Real.1
        mul_one_right(e)
        e * Real.1 = e
        c * d.inverse < e
        c / d = c * d.inverse
        c / d < e
    }
}

/// Adding a nonnegative number to a positive number keeps it positive.
theorem add_nonneg_pos_pos(a: Real, b: Real) {
    a >= Real.0 and b > Real.0 implies a + b > Real.0
} by {
    if a >= Real.0 and b > Real.0 {
        add_le_add_right(Real.0, a, b)
        Real.0 + b <= a + b
        add_zero_left(b)
        Real.0 + b = b
        b <= a + b
        lt_lte_trans(Real.0, b, a + b)
        Real.0 < a + b
    }
}

/// The sum of two positive numbers is positive.
theorem add_pos_pos(a: Real, b: Real) {
    a > Real.0 and b > Real.0 implies a + b > Real.0
} by {
    if a > Real.0 and b > Real.0 {
        lt_add_right(Real.0, a, b)
        b + Real.0 < b + a
        add_zero_right(b)
        b + Real.0 = b
        b < b + a
        lt_imp_lte(Real.0, b)
        Real.0 <= b
        lte_lt_trans(Real.0, b, b + a)
        Real.0 < b + a
        b + a = a + b
        Real.0 < a + b
    }
}

/// If a <= b and b is positive, then a / b <= 1.
theorem div_le_one(a: Real, b: Real) {
    a <= b and b > Real.0 implies a / b <= Real.1
} by {
    if a <= b and b > Real.0 {
        lt_imp_ne(Real.0, b)
        b != Real.0
        mul_div_cancel(a, b)
        b * (a / b) = a
        real_mul_comm(a / b, b)
        (a / b) * b = b * (a / b)
        (a / b) * b = a
        (a / b) * b <= b
        div_le_of_mul_le(a / b, b, b)
        a / b <= b / b
        b / b = b * b.inverse
        mul_inverse(b)
        b * b.inverse = Real.1
        b / b = Real.1
        a / b <= Real.1
    }
}

/// A strictly smaller absolute value implies different values.
theorem abs_lt_abs_imp_ne(a: Real, b: Real) {
    a.abs < b.abs implies a != b
} by {
    if a.abs < b.abs {
        lt_imp_ne(a.abs, b.abs)
        a.abs != b.abs
        a != b
    }
}

/// If f(x) exceeds c.abs / eta then c / f(x) is smaller than eta in absolute
/// value, and f(x) is nonzero.
theorem div_small_of_gt(c: Real, fx: Real, eta: Real) {
    c.abs / eta < fx and eta.is_positive
    implies fx != Real.0 and (c / fx).abs < eta
} by {
    if c.abs / eta < fx and eta.is_positive {
        pos_gt_zero(eta)
        eta > Real.0
        lt_imp_ne(Real.0, eta)
        Real.0 != eta
        eta != Real.0
        abs_gte_zero(c)
        c.abs >= Real.0
        div_nonneg_of_nonneg_pos(c.abs, eta)
        c.abs / eta >= Real.0
        lte_lt_trans(Real.0, c.abs / eta, fx)
        Real.0 < fx
        gt_zero_imp_pos(fx)
        fx.is_positive
        lt_imp_ne(Real.0, fx)
        Real.0 != fx
        fx != Real.0
        mul_lt_mul_of_pos_right[Real](c.abs / eta, fx, eta)
        (c.abs / eta) * eta < fx * eta
        mul_div_cancel(c.abs, eta)
        eta * (c.abs / eta) = c.abs
        real_mul_comm(c.abs / eta, eta)
        (c.abs / eta) * eta = eta * (c.abs / eta)
        (c.abs / eta) * eta = c.abs
        c.abs < fx * eta
        real_mul_comm(fx, eta)
        fx * eta = eta * fx
        c.abs < eta * fx
        lt_mul_pos_imp_div_lt(c.abs, fx, eta)
        c.abs / fx < eta
        abs_div(c, fx)
        (c / fx).abs = c.abs / fx.abs
        pos_imp_eq_abs(fx)
        fx = fx.abs
        c.abs / fx.abs = c.abs / fx
        (c / fx).abs = c.abs / fx
        (c / fx).abs < eta
        fx != Real.0 and (c / fx).abs < eta
    }
}

/// From x + y >= z follows x >= z - y.
theorem add_ge_imp_ge_sub(a: Real, b: Real, c: Real) {
    a + b >= c implies a >= c - b
} by {
    add_le_add_right(c, a + b, -b)
    c + -b <= a + b + -b
    sub_cancels(a, b)
    a + b + -b = a
    c + -b <= a
    a >= c + -b
    c - b = c + -b
    a >= c - b
}

/// The absolute value of one minus t is at least one minus the absolute value
/// of t.
theorem abs_one_minus_ge(t: Real) {
    (Real.1 - t).abs >= Real.1 - t.abs
} by {
    triangle_ineq(Real.1 - t, t)
    ((Real.1 - t) + t).abs <= (Real.1 - t).abs + t.abs
    (Real.1 - t) + t = Real.1
    Real.1.abs <= (Real.1 - t).abs + t.abs
    Real.1.is_positive
    pos_imp_eq_abs(Real.1)
    Real.1 = Real.1.abs
    Real.1.abs = Real.1
    Real.1 <= (Real.1 - t).abs + t.abs
    add_ge_imp_ge_sub((Real.1 - t).abs, t.abs, Real.1)
    (Real.1 - t).abs >= Real.1 - t.abs
}

// =============================================================================
// The Cauchy mean value ratio and the identity f/g = R * S
// =============================================================================

/// The ratio (f(b) - f(a)) / (g(b) - g(a)) equals df(c)/dg(c) at the Cauchy
/// mean value theorem point, when dg and the denominator stay nonzero.
theorem cauchy_mvt_ratio(f: Real -> Real, g: Real -> Real, df: Real -> Real,
    dg: Real -> Real, a: Real, b: Real) {
    continuous(f) and continuous(g) and a < b and is_derivative_fn(f, df) and
    is_derivative_fn(g, dg) and
    (forall(z: Real) { a < z and z < b implies dg(z) != Real.0 })
    implies exists(c: Real) {
        a < c and c < b and lhopital_ratio_x1(f, g, b, a) = df(c) / dg(c)
    }
} by {
    if continuous(f) and continuous(g) and a < b and is_derivative_fn(f, df) and
       is_derivative_fn(g, dg) and
       (forall(z: Real) { a < z and z < b implies dg(z) != Real.0 }) {
        // The denominator g(b) - g(a) is nonzero by the mean value theorem.
        mean_value_theorem(g, dg, a, b)
        let c2: Real satisfy {
            a < c2 and c2 < b and has_derivative_at(g, c2, secant_slope(g, a, b))
        }
        a < c2 and c2 < b
        has_derivative_at(g, c2, secant_slope(g, a, b))
        is_derivative_fn_at(g, dg, c2)
        has_derivative_at(g, c2, dg(c2))
        has_derivative_at_unique(g, c2, secant_slope(g, a, b), dg(c2))
        secant_slope(g, a, b) = dg(c2)
        forall(z: Real) { a < z and z < b implies dg(z) != Real.0 }
        a < c2 and c2 < b implies dg(c2) != Real.0
        dg(c2) != Real.0
        secant_slope(g, a, b) != Real.0
        secant_slope(g, a, b) = (g(b) - g(a)) / (b - a)
        (g(b) - g(a)) / (b - a) != Real.0
        lt_imp_ne(a, b)
        a != b
        sub_ne_zero_of_ne(b, a)
        b - a != Real.0
        div_mul_cancel_denominator(g(b) - g(a), b - a)
        ((g(b) - g(a)) / (b - a)) * (b - a) = g(b) - g(a)
        mul_not_zero[Real]((g(b) - g(a)) / (b - a), b - a)
        ((g(b) - g(a)) / (b - a)) * (b - a) != Real.0
        g(b) - g(a) != Real.0
        // Cauchy's mean value theorem gives the ratio point.
        cauchy_mean_value_theorem(f, g, df, dg, a, b)
        let c: Real satisfy {
            a < c and c < b and (f(b) - f(a)) * dg(c) = (g(b) - g(a)) * df(c)
        }
        a < c and c < b
        (f(b) - f(a)) * dg(c) = (g(b) - g(a)) * df(c)
        forall(z: Real) { a < z and z < b implies dg(z) != Real.0 }
        a < c and c < b implies dg(c) != Real.0
        dg(c) != Real.0
        div_cancel_common(f(b) - f(a), dg(c), g(b) - g(a))
        ((f(b) - f(a)) * dg(c)) / ((g(b) - g(a)) * dg(c)) =
            (f(b) - f(a)) / (g(b) - g(a))
        real_mul_comm(g(b) - g(a), df(c))
        (g(b) - g(a)) * df(c) = df(c) * (g(b) - g(a))
        (f(b) - f(a)) * dg(c) = df(c) * (g(b) - g(a))
        div_cancel_common(df(c), g(b) - g(a), dg(c))
        (df(c) * (g(b) - g(a))) / (dg(c) * (g(b) - g(a))) = df(c) / dg(c)
        real_mul_comm(g(b) - g(a), dg(c))
        (g(b) - g(a)) * dg(c) = dg(c) * (g(b) - g(a))
        ((f(b) - f(a)) * dg(c)) / ((g(b) - g(a)) * dg(c)) =
            (df(c) * (g(b) - g(a))) / (dg(c) * (g(b) - g(a)))
        (f(b) - f(a)) / (g(b) - g(a)) = df(c) / dg(c)
        lhopital_ratio_x1(f, g, b, a) = (f(b) - f(a)) / (g(b) - g(a))
        lhopital_ratio_x1(f, g, b, a) = df(c) / dg(c)
        exists(c3: Real) {
            a < c3 and c3 < b and lhopital_ratio_x1(f, g, b, a) = df(c3) / dg(c3)
        }
    }
}

/// The identity f(x)/g(x) = R(x) * S(x) for the two factors of the ∞/∞
/// proof, valid when the denominators are nonzero.
theorem lhopital_infinity_identity(f: Real -> Real, g: Real -> Real, x1: Real, x: Real) {
    f(x) != Real.0 and g(x) != Real.0 and f(x1) != f(x) and g(x1) != g(x)
    implies quotient_fn(f, g, x) = lhopital_ratio_x1(f, g, x1, x) * lhopital_scale_x1(f, g, x1, x)
} by {
    if f(x) != Real.0 and g(x) != Real.0 and f(x1) != f(x) and g(x1) != g(x) {
        // 1 - g(x1)/g(x) = (g(x) - g(x1))/g(x)
        mul_inverse(g(x))
        g(x) * g(x).inverse = Real.1
        g(x) / g(x) = g(x) * g(x).inverse
        g(x) / g(x) = Real.1
        div_sub_same_denom(Real.1, Real.1, g(x), g(x1), g(x))
        Real.1 * g(x) / g(x) - Real.1 * g(x1) / g(x) =
            (Real.1 * g(x) - Real.1 * g(x1)) / g(x)
        mul_one_left(g(x))
        Real.1 * g(x) = g(x)
        mul_one_left(g(x1))
        Real.1 * g(x1) = g(x1)
        Real.1 * g(x) / g(x) - Real.1 * g(x1) / g(x) =
            g(x) / g(x) - g(x1) / g(x)
        (Real.1 * g(x) - Real.1 * g(x1)) / g(x) = (g(x) - g(x1)) / g(x)
        g(x) / g(x) - g(x1) / g(x) = (g(x) - g(x1)) / g(x)
        Real.1 - g(x1) / g(x) = g(x) / g(x) - g(x1) / g(x)
        Real.1 - g(x1) / g(x) = (g(x) - g(x1)) / g(x)
        // 1 - f(x1)/f(x) = (f(x) - f(x1))/f(x)
        mul_inverse(f(x))
        f(x) * f(x).inverse = Real.1
        f(x) / f(x) = f(x) * f(x).inverse
        f(x) / f(x) = Real.1
        div_sub_same_denom(Real.1, Real.1, f(x), f(x1), f(x))
        Real.1 * f(x) / f(x) - Real.1 * f(x1) / f(x) =
            (Real.1 * f(x) - Real.1 * f(x1)) / f(x)
        mul_one_left(f(x))
        Real.1 * f(x) = f(x)
        mul_one_left(f(x1))
        Real.1 * f(x1) = f(x1)
        Real.1 * f(x) / f(x) - Real.1 * f(x1) / f(x) =
            f(x) / f(x) - f(x1) / f(x)
        (Real.1 * f(x) - Real.1 * f(x1)) / f(x) = (f(x) - f(x1)) / f(x)
        f(x) / f(x) - f(x1) / f(x) = (f(x) - f(x1)) / f(x)
        Real.1 - f(x1) / f(x) = f(x) / f(x) - f(x1) / f(x)
        Real.1 - f(x1) / f(x) = (f(x) - f(x1)) / f(x)
        // The scale factor, rewritten as a division of fractions.
        lhopital_scale_x1(f, g, x1, x) =
            (Real.1 - g(x1) / g(x)) / (Real.1 - f(x1) / f(x))
        lhopital_scale_x1(f, g, x1, x) =
            ((g(x) - g(x1)) / g(x)) / ((f(x) - f(x1)) / f(x))
        sub_ne_zero_of_ne(f(x), f(x1))
        f(x) - f(x1) != Real.0
        div_div(g(x) - g(x1), g(x), f(x) - f(x1), f(x))
        ((g(x) - g(x1)) / g(x)) / ((f(x) - f(x1)) / f(x)) =
            ((g(x) - g(x1)) * f(x)) / (g(x) * (f(x) - f(x1)))
        lhopital_scale_x1(f, g, x1, x) =
            ((g(x) - g(x1)) * f(x)) / (g(x) * (f(x) - f(x1)))
        // The minus signs cancel.
        g(x) - g(x1) = -(g(x1) - g(x))
        f(x) - f(x1) = -(f(x1) - f(x))
        mul_neg_left(g(x1) - g(x), f(x))
        -(g(x1) - g(x)) * f(x) = -((g(x1) - g(x)) * f(x))
        (g(x) - g(x1)) * f(x) = -((g(x1) - g(x)) * f(x))
        mul_neg_right(g(x), f(x1) - f(x))
        g(x) * (-(f(x1) - f(x))) = -(g(x) * (f(x1) - f(x)))
        g(x) * (f(x) - f(x1)) = -(g(x) * (f(x1) - f(x)))
        lhopital_scale_x1(f, g, x1, x) =
            (-((g(x1) - g(x)) * f(x))) / (-(g(x) * (f(x1) - f(x))))
        sub_ne_zero_of_ne(f(x1), f(x))
        f(x1) - f(x) != Real.0
        mul_not_zero[Real](g(x), f(x1) - f(x))
        g(x) * (f(x1) - f(x)) != Real.0
        neg_div_neg_eq((g(x1) - g(x)) * f(x), g(x) * (f(x1) - f(x)))
        (-((g(x1) - g(x)) * f(x))) / (-(g(x) * (f(x1) - f(x)))) =
            ((g(x1) - g(x)) * f(x)) / (g(x) * (f(x1) - f(x)))
        lhopital_scale_x1(f, g, x1, x) =
            ((g(x1) - g(x)) * f(x)) / (g(x) * (f(x1) - f(x)))
        // The product R * S telescopes to f(x)/g(x).
        sub_ne_zero_of_ne(g(x1), g(x))
        g(x1) - g(x) != Real.0
        mul_not_zero[Real](g(x), f(x1) - f(x))
        g(x) * (f(x1) - f(x)) != Real.0
        mul_div(f(x1) - f(x), g(x1) - g(x), (g(x1) - g(x)) * f(x),
            g(x) * (f(x1) - f(x)))
        ((f(x1) - f(x)) / (g(x1) - g(x))) *
            (((g(x1) - g(x)) * f(x)) / (g(x) * (f(x1) - f(x)))) =
            ((f(x1) - f(x)) * ((g(x1) - g(x)) * f(x))) /
            ((g(x1) - g(x)) * (g(x) * (f(x1) - f(x))))
        lhopital_ratio_x1(f, g, x1, x) = (f(x1) - f(x)) / (g(x1) - g(x))
        lhopital_ratio_x1(f, g, x1, x) * lhopital_scale_x1(f, g, x1, x) =
            ((f(x1) - f(x)) * ((g(x1) - g(x)) * f(x))) /
            ((g(x1) - g(x)) * (g(x) * (f(x1) - f(x))))
        // Cancel the common factor (g(x1) - g(x)).
        mul_assoc(g(x1) - g(x), g(x), f(x1) - f(x))
        ((g(x1) - g(x)) * g(x)) * (f(x1) - f(x)) =
            (g(x1) - g(x)) * (g(x) * (f(x1) - f(x)))
        real_mul_comm(g(x1) - g(x), g(x))
        (g(x1) - g(x)) * g(x) = g(x) * (g(x1) - g(x))
        (g(x) * (g(x1) - g(x))) * (f(x1) - f(x)) =
            (g(x1) - g(x)) * (g(x) * (f(x1) - f(x)))
        mul_assoc(f(x1) - f(x), g(x1) - g(x), f(x))
        ((f(x1) - f(x)) * (g(x1) - g(x))) * f(x) =
            (f(x1) - f(x)) * ((g(x1) - g(x)) * f(x))
        mul_assoc(f(x1) - f(x), f(x), g(x1) - g(x))
        ((f(x1) - f(x)) * f(x)) * (g(x1) - g(x)) =
            (f(x1) - f(x)) * (f(x) * (g(x1) - g(x)))
        real_mul_comm(f(x), g(x1) - g(x))
        f(x) * (g(x1) - g(x)) = (g(x1) - g(x)) * f(x)
        ((f(x1) - f(x)) * f(x)) * (g(x1) - g(x)) =
            (f(x1) - f(x)) * ((g(x1) - g(x)) * f(x))
        // The quotient with the (g(x1) - g(x)) factor cancelled:
        // numerator ((f(x1)-f(x))*f(x))*(g(x1)-g(x)), denominator
        // (g(x)*(g(x1)-g(x)))*(f(x1)-f(x)).
        div_cancel_common((f(x1) - f(x)) * f(x), g(x1) - g(x),
            g(x) * (f(x1) - f(x)))
        (((f(x1) - f(x)) * f(x)) * (g(x1) - g(x))) /
            ((g(x) * (f(x1) - f(x))) * (g(x1) - g(x))) =
            ((f(x1) - f(x)) * f(x)) / (g(x) * (f(x1) - f(x)))
        real_mul_comm(g(x) * (f(x1) - f(x)), g(x1) - g(x))
        (g(x) * (f(x1) - f(x))) * (g(x1) - g(x)) =
            (g(x1) - g(x)) * (g(x) * (f(x1) - f(x)))
        (((f(x1) - f(x)) * f(x)) * (g(x1) - g(x))) /
            ((g(x1) - g(x)) * (g(x) * (f(x1) - f(x)))) =
            ((f(x1) - f(x)) * f(x)) / (g(x) * (f(x1) - f(x)))
        // Cancel the common factor (f(x1) - f(x)).
        real_mul_comm(f(x1) - f(x), f(x))
        (f(x1) - f(x)) * f(x) = f(x) * (f(x1) - f(x))
        div_cancel_common(f(x), f(x1) - f(x), g(x))
        (f(x) * (f(x1) - f(x))) / (g(x) * (f(x1) - f(x))) = f(x) / g(x)
        ((f(x1) - f(x)) * f(x)) / (g(x) * (f(x1) - f(x))) = f(x) / g(x)
        lhopital_ratio_x1(f, g, x1, x) * lhopital_scale_x1(f, g, x1, x) =
            f(x) / g(x)
        quotient_fn(f, g, x) = f(x) / g(x)
        quotient_fn(f, g, x) =
            lhopital_ratio_x1(f, g, x1, x) * lhopital_scale_x1(f, g, x1, x)
    }
}

/// The fixed-endpoint ratio is close to the derivative-ratio limit when the
/// derivative ratio is close to it on the whole neighborhood (a, a + delta1).
theorem lhopital_infinity_ratio_pointwise(f: Real -> Real, g: Real -> Real,
    df: Real -> Real, dg: Real -> Real, a: Real, b: Real, x1: Real, delta1: Real,
    l: Real, eps1: Real, x: Real) {
    continuous(f) and continuous(g) and is_derivative_fn(f, df) and
    is_derivative_fn(g, dg) and
    (forall(z: Real) { a < z and z < b implies dg(z) != Real.0 }) and
    delta1.is_positive and a < x1 and x1 < a + delta1 and a + delta1 < b and
    (forall(z: Real) { a < z and z < a + delta1 implies quotient_fn(df, dg, z).is_close(l, eps1) }) and
    a < x and x < x1
    implies lhopital_ratio_x1(f, g, x1, x).is_close(l, eps1)
} by {
    if continuous(f) and continuous(g) and is_derivative_fn(f, df) and
       is_derivative_fn(g, dg) and
       (forall(z: Real) { a < z and z < b implies dg(z) != Real.0 }) and
       delta1.is_positive and a < x1 and x1 < a + delta1 and a + delta1 < b and
       (forall(z: Real) { a < z and z < a + delta1 implies quotient_fn(df, dg, z).is_close(l, eps1) }) and
       a < x and x < x1 {
        // dg stays nonzero on (x, x1).
        lt_trans[Real](x1, a + delta1, b)
        x1 < b
        forall(z: Real) {
            if x < z and z < x1 {
                lt_trans[Real](a, x, z)
                a < z
                lt_trans[Real](z, x1, b)
                z < b
                forall(z0: Real) { a < z0 and z0 < b implies dg(z0) != Real.0 }
                a < z and z < b implies dg(z) != Real.0
                dg(z) != Real.0
            }
        }
        cauchy_mvt_ratio(f, g, df, dg, x, x1)
        let c: Real satisfy {
            x < c and c < x1 and lhopital_ratio_x1(f, g, x1, x) = df(c) / dg(c)
        }
        x < c and c < x1
        lhopital_ratio_x1(f, g, x1, x) = df(c) / dg(c)
        // The Cauchy point c lies in (a, a + delta1).
        lt_trans[Real](a, x, c)
        a < c
        lt_trans[Real](c, x1, a + delta1)
        c < a + delta1
        forall(z: Real) { a < z and z < a + delta1 implies quotient_fn(df, dg, z).is_close(l, eps1) }
        a < c and c < a + delta1 implies quotient_fn(df, dg, c).is_close(l, eps1)
        quotient_fn(df, dg, c).is_close(l, eps1)
        quotient_fn(df, dg, c) = df(c) / dg(c)
        lhopital_ratio_x1(f, g, x1, x).is_close(l, eps1)
    }
}

/// A nonzero absolute value implies the value itself is nonzero.
theorem abs_ne_zero_imp_ne(a: Real) {
    a.abs != Real.0 implies a != Real.0
} by {
    if a.abs != Real.0 {
        if a = Real.0 {
            a.abs = Real.0
            false
        }
        a != Real.0
    }
}

/// The scale factor S(x) is close to 1 when both quotients f(x1)/f(x) and
/// g(x1)/g(x) are small in absolute value.
theorem lhopital_infinity_scale_pointwise(f: Real -> Real, g: Real -> Real,
    x1: Real, x: Real, eta: Real) {
    f(x) != Real.0 and g(x) != Real.0 and
    (f(x1) / f(x)).abs < eta and (g(x1) / g(x)).abs < eta and
    eta.is_positive and eta <= Real.one_half
    implies lhopital_scale_x1(f, g, x1, x).is_close(Real.1, (two * two) * eta)
} by {
    if f(x) != Real.0 and g(x) != Real.0 and
       (f(x1) / f(x)).abs < eta and (g(x1) / g(x)).abs < eta and
       eta.is_positive and eta <= Real.one_half {
        // eta is positive and less than 1; one_half is positive.
        pos_gt_zero(eta)
        eta > Real.0
        one_half_positive
        Real.0 < Real.one_half
        gt_zero_imp_pos(Real.one_half)
        Real.one_half.is_positive
        lt_add_pos(Real.one_half, Real.one_half)
        Real.one_half < Real.one_half + Real.one_half
        one_half_plus_one_half
        Real.one_half + Real.one_half = Real.1
        Real.one_half < Real.1
        lte_lt_trans(eta, Real.one_half, Real.1)
        eta < Real.1
        // |1 - t| >= 1 - |t| > 1 - eta >= one_half > 0.
        abs_one_minus_ge(f(x1) / f(x))
        (Real.1 - f(x1) / f(x)).abs >= Real.1 - (f(x1) / f(x)).abs
        sub_lt_is_gt(Real.1, (f(x1) / f(x)).abs, eta)
        Real.1 - (f(x1) / f(x)).abs > Real.1 - eta
        lte_imp_neg_lte_neg(eta, Real.one_half)
        -Real.one_half <= -eta
        add_le_add_right(-Real.one_half, -eta, Real.1)
        -Real.one_half + Real.1 <= -eta + Real.1
        -Real.one_half + Real.1 = Real.1 - Real.one_half
        Real.1 - Real.one_half = Real.one_half
        Real.one_half <= Real.1 - eta
        lte_lt_trans(Real.one_half, Real.1 - eta, Real.1 - (f(x1) / f(x)).abs)
        Real.one_half < Real.1 - (f(x1) / f(x)).abs
        lt_imp_lte(Real.one_half, Real.1 - (f(x1) / f(x)).abs)
        Real.one_half <= Real.1 - (f(x1) / f(x)).abs
        Real.1 - (f(x1) / f(x)).abs <= (Real.1 - f(x1) / f(x)).abs
        lt_lte_trans(Real.one_half, Real.1 - (f(x1) / f(x)).abs,
            (Real.1 - f(x1) / f(x)).abs)
        Real.one_half < (Real.1 - f(x1) / f(x)).abs
        lt_imp_lte(Real.one_half, (Real.1 - f(x1) / f(x)).abs)
        Real.one_half <= (Real.1 - f(x1) / f(x)).abs
        lte_lt_trans(Real.0, Real.one_half, (Real.1 - f(x1) / f(x)).abs)
        Real.0 < (Real.1 - f(x1) / f(x)).abs
        lt_imp_ne(Real.0, (Real.1 - f(x1) / f(x)).abs)
        Real.0 != (Real.1 - f(x1) / f(x)).abs
        (Real.1 - f(x1) / f(x)).abs != Real.0
        abs_ne_zero_imp_ne(Real.1 - f(x1) / f(x))
        Real.1 - f(x1) / f(x) != Real.0
        // S - 1 = (t - u) / (1 - t).
        mul_inverse(Real.1 - f(x1) / f(x))
        (Real.1 - f(x1) / f(x)) * (Real.1 - f(x1) / f(x)).inverse = Real.1
        (Real.1 - f(x1) / f(x)) / (Real.1 - f(x1) / f(x)) =
            (Real.1 - f(x1) / f(x)) * (Real.1 - f(x1) / f(x)).inverse
        (Real.1 - f(x1) / f(x)) / (Real.1 - f(x1) / f(x)) = Real.1
        div_sub_same_denom(Real.1, Real.1, Real.1 - g(x1) / g(x),
            Real.1 - f(x1) / f(x), Real.1 - f(x1) / f(x))
        Real.1 * (Real.1 - g(x1) / g(x)) / (Real.1 - f(x1) / f(x)) -
            Real.1 * (Real.1 - f(x1) / f(x)) / (Real.1 - f(x1) / f(x)) =
            (Real.1 * (Real.1 - g(x1) / g(x)) -
                Real.1 * (Real.1 - f(x1) / f(x))) /
            (Real.1 - f(x1) / f(x))
        mul_one_left(Real.1 - g(x1) / g(x))
        Real.1 * (Real.1 - g(x1) / g(x)) = Real.1 - g(x1) / g(x)
        mul_one_left(Real.1 - f(x1) / f(x))
        Real.1 * (Real.1 - f(x1) / f(x)) = Real.1 - f(x1) / f(x)
        (Real.1 - g(x1) / g(x)) / (Real.1 - f(x1) / f(x)) -
            (Real.1 - f(x1) / f(x)) / (Real.1 - f(x1) / f(x)) =
            ((Real.1 - g(x1) / g(x)) - (Real.1 - f(x1) / f(x))) /
            (Real.1 - f(x1) / f(x))
        lhopital_scale_x1(f, g, x1, x) =
            (Real.1 - g(x1) / g(x)) / (Real.1 - f(x1) / f(x))
        lhopital_scale_x1(f, g, x1, x) - Real.1 =
            ((Real.1 - g(x1) / g(x)) - (Real.1 - f(x1) / f(x))) /
            (Real.1 - f(x1) / f(x))
        (Real.1 - g(x1) / g(x)) - (Real.1 - f(x1) / f(x)) =
            f(x1) / f(x) - g(x1) / g(x)
        lhopital_scale_x1(f, g, x1, x) - Real.1 =
            (f(x1) / f(x) - g(x1) / g(x)) / (Real.1 - f(x1) / f(x))
        // |S - 1| = |t - u| / |1 - t| <= (|t| + |u|) / |1 - t|.
        abs_div(f(x1) / f(x) - g(x1) / g(x), Real.1 - f(x1) / f(x))
        ((f(x1) / f(x) - g(x1) / g(x)) / (Real.1 - f(x1) / f(x))).abs =
            (f(x1) / f(x) - g(x1) / g(x)).abs /
            (Real.1 - f(x1) / f(x)).abs
        (lhopital_scale_x1(f, g, x1, x) - Real.1).abs =
            (f(x1) / f(x) - g(x1) / g(x)).abs /
            (Real.1 - f(x1) / f(x)).abs
        triangle_ineq(f(x1) / f(x), -(g(x1) / g(x)))
        (f(x1) / f(x) + -(g(x1) / g(x))).abs <= (f(x1) / f(x)).abs + (-(g(x1) / g(x))).abs
        f(x1) / f(x) - g(x1) / g(x) = f(x1) / f(x) + -(g(x1) / g(x))
        (f(x1) / f(x) - g(x1) / g(x)).abs <= (f(x1) / f(x)).abs + (-(g(x1) / g(x))).abs
        abs_neg(g(x1) / g(x))
        (-(g(x1) / g(x))).abs = (g(x1) / g(x)).abs
        (f(x1) / f(x) - g(x1) / g(x)).abs <= (f(x1) / f(x)).abs + (g(x1) / g(x)).abs
        div_le_div_pos((f(x1) / f(x) - g(x1) / g(x)).abs,
            (f(x1) / f(x)).abs + (g(x1) / g(x)).abs,
            (Real.1 - f(x1) / f(x)).abs)
        (f(x1) / f(x) - g(x1) / g(x)).abs /
            (Real.1 - f(x1) / f(x)).abs <= ((f(x1) / f(x)).abs + (g(x1) / g(x)).abs) /
            (Real.1 - f(x1) / f(x)).abs
        (lhopital_scale_x1(f, g, x1, x) - Real.1).abs <= ((f(x1) / f(x)).abs + (g(x1) / g(x)).abs) /
            (Real.1 - f(x1) / f(x)).abs
        // The numerator is < 2 * eta and the denominator is > one_half.
        add_lt_lt((f(x1) / f(x)).abs, eta, (g(x1) / g(x)).abs, eta)
        (f(x1) / f(x)).abs + (g(x1) / g(x)).abs < eta + eta
        div_lt_div_pos((f(x1) / f(x)).abs + (g(x1) / g(x)).abs, eta + eta,
            (Real.1 - f(x1) / f(x)).abs)
        ((f(x1) / f(x)).abs + (g(x1) / g(x)).abs) /
            (Real.1 - f(x1) / f(x)).abs < (eta + eta) / (Real.1 - f(x1) / f(x)).abs
        lte_lt_trans((lhopital_scale_x1(f, g, x1, x) - Real.1).abs,
            ((f(x1) / f(x)).abs + (g(x1) / g(x)).abs) /
                (Real.1 - f(x1) / f(x)).abs,
            (eta + eta) / (Real.1 - f(x1) / f(x)).abs)
        (lhopital_scale_x1(f, g, x1, x) - Real.1).abs < (eta + eta) / (Real.1 - f(x1) / f(x)).abs
        // (eta + eta) / |1 - t| < (eta + eta) / one_half = 4 * eta.
        add_pos_pos(eta, eta)
        eta + eta > Real.0
        gt_zero_imp_pos(eta + eta)
        (eta + eta).is_positive
        inverse_of_positive_is_positive[Real](Real.one_half)
        Real.0 < Real.one_half
        Real.one_half.inverse > Real.0
        inverse_on_positive_flips_inequality[Real](Real.one_half,
            (Real.1 - f(x1) / f(x)).abs)
        (Real.1 - f(x1) / f(x)).abs.inverse < Real.one_half.inverse
        mul_lt_mul_of_pos_right[Real]((Real.1 - f(x1) / f(x)).abs.inverse,
            Real.one_half.inverse, eta + eta)
        (Real.1 - f(x1) / f(x)).abs.inverse * (eta + eta) < Real.one_half.inverse * (eta + eta)
        (eta + eta) * (Real.1 - f(x1) / f(x)).abs.inverse < (eta + eta) * Real.one_half.inverse
        (eta + eta) / (Real.1 - f(x1) / f(x)).abs =
            (eta + eta) * (Real.1 - f(x1) / f(x)).abs.inverse
        (eta + eta) / Real.one_half = (eta + eta) * Real.one_half.inverse
        (eta + eta) / (Real.1 - f(x1) / f(x)).abs < (eta + eta) / Real.one_half
        lt_trans[Real]((lhopital_scale_x1(f, g, x1, x) - Real.1).abs,
            (eta + eta) / (Real.1 - f(x1) / f(x)).abs,
            (eta + eta) / Real.one_half)
        (lhopital_scale_x1(f, g, x1, x) - Real.1).abs < (eta + eta) / Real.one_half
        // (eta + eta) / one_half = (two * two) * eta.
        mul_distrib_left(eta, Real.1, Real.1)
        (Real.1 + Real.1) * eta = Real.1 * eta + Real.1 * eta
        two * eta = Real.1 * eta + Real.1 * eta
        mul_one_left(eta)
        Real.1 * eta = eta
        two * eta = eta + eta
        eta + eta = two * eta
        mul_distrib_left(Real.one_half, Real.1, Real.1)
        (Real.1 + Real.1) * Real.one_half =
            Real.1 * Real.one_half + Real.1 * Real.one_half
        two * Real.one_half = Real.1 * Real.one_half + Real.1 * Real.one_half
        mul_one_left(Real.one_half)
        Real.1 * Real.one_half = Real.one_half
        two * Real.one_half = Real.one_half + Real.one_half
        one_half_plus_one_half
        Real.one_half + Real.one_half = Real.1
        two * Real.one_half = Real.1
        real_mul_comm(Real.one_half, two)
        Real.one_half * two = two * Real.one_half
        Real.one_half * two = Real.1
        lt_imp_ne(Real.0, Real.one_half)
        Real.one_half != Real.0
        mul_left_cancel(Real.1, Real.one_half, two)
        Real.1 / Real.one_half = two
        Real.one_half.inverse = Real.1 / Real.one_half
        Real.one_half.inverse = two
        two * eta / Real.one_half = two * eta * Real.one_half.inverse
        two * eta * Real.one_half.inverse = two * eta * two
        real_mul_comm(two * eta, two)
        two * eta * two = two * (two * eta)
        two * (two * eta) = (two * two) * eta
        two * eta * Real.one_half.inverse = (two * two) * eta
        two * eta / Real.one_half = (two * two) * eta
        (eta + eta) / Real.one_half = (two * two) * eta
        (lhopital_scale_x1(f, g, x1, x) - Real.1).abs < (two * two) * eta
        lhopital_scale_x1(f, g, x1, x).is_close(Real.1, (two * two) * eta)
    }
}

// =============================================================================
// The ε-δ core and the two final forms of the rule
// =============================================================================

/// The ε-δ core of the ∞/∞ rule: for every tolerance eps there is a right
/// neighborhood of a on which |f(x)/g(x) - l| < eps.
theorem lhopital_infinity_core(f: Real -> Real, g: Real -> Real, df: Real -> Real,
    dg: Real -> Real, a: Real, b: Real, l: Real, eps: Real) {
    continuous(f) and continuous(g) and is_derivative_fn(f, df) and
    is_derivative_fn(g, dg) and
    (forall(x: Real) { a < x and x < b implies dg(x) != Real.0 }) and
    limit_at_right(quotient_fn(df, dg), a, b, l) and
    tends_infinity_right(f, a, b) and tends_infinity_right(g, a, b) and
    eps.is_positive
    implies exists(delta: Real) {
        delta.is_positive and a + delta < b and
        forall(x: Real) {
            a < x and x < a + delta implies quotient_fn(f, g, x).is_close(l, eps)
        }
    }
} by {
    if continuous(f) and continuous(g) and is_derivative_fn(f, df) and
       is_derivative_fn(g, dg) and
       (forall(x: Real) { a < x and x < b implies dg(x) != Real.0 }) and
       limit_at_right(quotient_fn(df, dg), a, b, l) and
       tends_infinity_right(f, a, b) and tends_infinity_right(g, a, b) and
       eps.is_positive {
        // Choose eps1 with eps1 + eps1 + eps1 < eps.
        find_less_than_a_third(eps)
        let eps1: Real satisfy {
            eps1.is_positive and eps1 + eps1 + eps1 < eps
        }
        eps1.is_positive
        gt_zero_imp_pos(eps1)
        eps1 > Real.0
        // The derivative-ratio limit at eps1 gives delta1.
        limit_at_right_apply(quotient_fn(df, dg), a, b, l, eps1)
        let delta1: Real satisfy {
            delta1.is_positive and a + delta1 < b and
            forall(x2: Real) {
                a < x2 and x2 < a + delta1 implies quotient_fn(df, dg, x2).is_close(l, eps1)
            }
        }
        delta1.is_positive
        gt_zero_imp_pos(delta1)
        delta1 > Real.0
        a + delta1 < b
        // The fixed endpoint x1 = a + delta1 * one_half.
        one_half_positive
        Real.0 < Real.one_half
        gt_zero_imp_pos(Real.one_half)
        Real.one_half.is_positive
        mul_pos_pos(delta1, Real.one_half)
        (delta1 * Real.one_half).is_positive
        gt_zero_imp_pos(delta1 * Real.one_half)
        delta1 * Real.one_half > Real.0
        lt_add_pos(a, delta1 * Real.one_half)
        a < a + delta1 * Real.one_half
        mul_distrib_right(delta1, Real.one_half, Real.one_half)
        delta1 * (Real.one_half + Real.one_half) = delta1 * Real.one_half + delta1 * Real.one_half
        one_half_plus_one_half
        Real.one_half + Real.one_half = Real.1
        delta1 * Real.one_half + delta1 * Real.one_half = delta1 * Real.1
        mul_one_right(delta1)
        delta1 * Real.1 = delta1
        delta1 * Real.one_half + delta1 * Real.one_half = delta1
        lt_add_pos(delta1 * Real.one_half, delta1 * Real.one_half)
        delta1 * Real.one_half < delta1 * Real.one_half + delta1 * Real.one_half
        delta1 * Real.one_half < delta1
        lt_add_right(delta1 * Real.one_half, delta1, a)
        a + delta1 * Real.one_half < a + delta1
        lt_trans[Real](a + delta1 * Real.one_half, a + delta1, b)
        a + delta1 * Real.one_half < b
        // The tolerance eps2 = eps1 / (l.abs + eps1 + 1) for the scale.
        abs_gte_zero(l)
        l.abs >= Real.0
        add_nonneg_pos_pos(l.abs, eps1)
        l.abs + eps1 > Real.0
        Real.1.is_positive
        gt_zero_imp_pos(Real.1)
        Real.1 > Real.0
        add_pos_pos(l.abs + eps1, Real.1)
        l.abs + eps1 + Real.1 > Real.0
        let eps2 = eps1 / (l.abs + eps1 + Real.1)
        div_pos_of_pos_pos(eps1, l.abs + eps1 + Real.1)
        eps2 > Real.0
        gt_zero_imp_pos(eps2)
        eps2.is_positive
        // eps2 <= 1.
        lt_add_pos(eps1, l.abs + Real.1)
        eps1 < eps1 + (l.abs + Real.1)
        eps1 + (l.abs + Real.1) = l.abs + eps1 + Real.1
        eps1 < l.abs + eps1 + Real.1
        lt_imp_lte(eps1, l.abs + eps1 + Real.1)
        eps1 <= l.abs + eps1 + Real.1
        div_le_one(eps1, l.abs + eps1 + Real.1)
        eps2 <= Real.1
        // The small-scale tolerance eta = eps2 * one_half * one_half.
        mul_pos_pos(eps2, Real.one_half)
        (eps2 * Real.one_half).is_positive
        mul_pos_pos(eps2 * Real.one_half, Real.one_half)
        ((eps2 * Real.one_half) * Real.one_half).is_positive
        let eta = (eps2 * Real.one_half) * Real.one_half
        gt_zero_imp_pos(eta)
        eta > Real.0
        // eta <= one_half.
        mul_pos_pos(Real.one_half, Real.one_half)
        (Real.one_half * Real.one_half).is_positive
        gt_zero_imp_pos(Real.one_half * Real.one_half)
        Real.one_half * Real.one_half > Real.0
        mul_le_mul_pos_right(eps2, Real.1, Real.one_half * Real.one_half)
        eps2 * (Real.one_half * Real.one_half) <= Real.1 * (Real.one_half * Real.one_half)
        mul_one_left(Real.one_half * Real.one_half)
        Real.1 * (Real.one_half * Real.one_half) = Real.one_half * Real.one_half
        eps2 * (Real.one_half * Real.one_half) <= Real.one_half * Real.one_half
        lt_add_pos(Real.one_half, Real.one_half)
        Real.one_half < Real.one_half + Real.one_half
        Real.one_half + Real.one_half = Real.1
        Real.one_half < Real.1
        lt_imp_lte(Real.one_half, Real.1)
        Real.one_half <= Real.1
        mul_le_mul_pos_right(Real.one_half, Real.1, Real.one_half)
        Real.one_half * Real.one_half <= Real.1 * Real.one_half
        mul_one_left(Real.one_half)
        Real.1 * Real.one_half = Real.one_half
        Real.one_half * Real.one_half <= Real.one_half
        mul_assoc(eps2, Real.one_half, Real.one_half)
        (eps2 * Real.one_half) * Real.one_half = eps2 * (Real.one_half * Real.one_half)
        eta <= Real.one_half * Real.one_half
        lte_trans(eta, Real.one_half * Real.one_half, Real.one_half)
        eta <= Real.one_half
        // The deltas from tends to +infinity at the thresholds |f(x1)|/eta.
        tends_infinity_right_apply(f, a, b, f(a + delta1 * Real.one_half).abs / eta)
        let delta_f: Real satisfy {
            delta_f.is_positive and a + delta_f < b and
            forall(u: Real) {
                a < u and u < a + delta_f implies f(a + delta1 * Real.one_half).abs / eta < f(u)
            }
        }
        delta_f.is_positive
        tends_infinity_right_apply(g, a, b, g(a + delta1 * Real.one_half).abs / eta)
        let delta_g: Real satisfy {
            delta_g.is_positive and a + delta_g < b and
            forall(u: Real) {
                a < u and u < a + delta_g implies g(a + delta1 * Real.one_half).abs / eta < g(u)
            }
        }
        delta_g.is_positive
        // The common delta, smaller than all the pieces.
        eps_smaller_than_both(delta1 * Real.one_half, delta_f)
        let delta2: Real satisfy {
            delta2.is_positive and delta2 < delta1 * Real.one_half and delta2 < delta_f
        }
        delta2.is_positive
        eps_smaller_than_both(delta2, delta_g)
        let delta: Real satisfy {
            delta.is_positive and delta < delta2 and delta < delta_g
        }
        delta.is_positive
        lt_trans[Real](delta, delta2, delta1 * Real.one_half)
        delta < delta1 * Real.one_half
        lt_trans[Real](delta, delta2, delta_f)
        delta < delta_f
        delta < delta_g
        lt_add_right(delta, delta1 * Real.one_half, a)
        a + delta < a + delta1 * Real.one_half
        lt_trans[Real](a + delta, a + delta1 * Real.one_half, b)
        a + delta < b
        lt_add_right(delta, delta_f, a)
        a + delta < a + delta_f
        lt_add_right(delta, delta_g, a)
        a + delta < a + delta_g
        forall(x: Real) {
            if a < x and x < a + delta {
                // x < x1 and x is inside the f- and g-neighborhoods.
                lt_trans[Real](x, a + delta, a + delta1 * Real.one_half)
                x < a + delta1 * Real.one_half
                lt_trans[Real](x, a + delta, a + delta_f)
                x < a + delta_f
                lt_trans[Real](x, a + delta, a + delta_g)
                x < a + delta_g
                // The small quotients for f and g.
                forall(u: Real) {
                    a < u and u < a + delta_f implies f(a + delta1 * Real.one_half).abs / eta < f(u)
                }
                a < x and x < a + delta_f implies f(a + delta1 * Real.one_half).abs / eta < f(x)
                f(a + delta1 * Real.one_half).abs / eta < f(x)
                div_small_of_gt(f(a + delta1 * Real.one_half), f(x), eta)
                f(x) != Real.0
                (f(a + delta1 * Real.one_half) / f(x)).abs < eta
                forall(u: Real) {
                    a < u and u < a + delta_g implies g(a + delta1 * Real.one_half).abs / eta < g(u)
                }
                a < x and x < a + delta_g implies g(a + delta1 * Real.one_half).abs / eta < g(x)
                g(a + delta1 * Real.one_half).abs / eta < g(x)
                div_small_of_gt(g(a + delta1 * Real.one_half), g(x), eta)
                g(x) != Real.0
                (g(a + delta1 * Real.one_half) / g(x)).abs < eta
                // f(x) and g(x) are positive.
                abs_gte_zero(f(a + delta1 * Real.one_half))
                f(a + delta1 * Real.one_half).abs >= Real.0
                div_nonneg_of_nonneg_pos(f(a + delta1 * Real.one_half).abs, eta)
                f(a + delta1 * Real.one_half).abs / eta >= Real.0
                lte_lt_trans(Real.0, f(a + delta1 * Real.one_half).abs / eta, f(x))
                Real.0 < f(x)
                gt_zero_imp_pos(f(x))
                f(x).is_positive
                abs_gte_zero(g(a + delta1 * Real.one_half))
                g(a + delta1 * Real.one_half).abs >= Real.0
                div_nonneg_of_nonneg_pos(g(a + delta1 * Real.one_half).abs, eta)
                g(a + delta1 * Real.one_half).abs / eta >= Real.0
                lte_lt_trans(Real.0, g(a + delta1 * Real.one_half).abs / eta, g(x))
                Real.0 < g(x)
                gt_zero_imp_pos(g(x))
                g(x).is_positive
                // The identity factors are well-defined: f(x1) != f(x), g(x1) != g(x).
                abs_div(f(a + delta1 * Real.one_half), f(x))
                (f(a + delta1 * Real.one_half) / f(x)).abs =
                    f(a + delta1 * Real.one_half).abs / f(x).abs
                pos_imp_eq_abs(f(x))
                f(x).is_positive
                f(x) = f(x).abs
                (f(a + delta1 * Real.one_half) / f(x)).abs < eta
                f(a + delta1 * Real.one_half).abs / f(x).abs < eta
                div_lt_mul_pos(f(a + delta1 * Real.one_half).abs, f(x).abs, eta)
                f(a + delta1 * Real.one_half).abs < eta * f(x).abs
                lte_lt_trans(eta, Real.one_half, Real.1)
                eta < Real.1
                mul_lt_mul_of_pos_right[Real](eta, Real.1, f(x).abs)
                eta * f(x).abs < Real.1 * f(x).abs
                mul_one_left(f(x).abs)
                Real.1 * f(x).abs = f(x).abs
                eta * f(x).abs < f(x).abs
                lt_trans[Real](f(a + delta1 * Real.one_half).abs, eta * f(x).abs, f(x).abs)
                f(a + delta1 * Real.one_half).abs < f(x).abs
                abs_lt_abs_imp_ne(f(a + delta1 * Real.one_half), f(x))
                f(a + delta1 * Real.one_half) != f(x)
                abs_div(g(a + delta1 * Real.one_half), g(x))
                (g(a + delta1 * Real.one_half) / g(x)).abs =
                    g(a + delta1 * Real.one_half).abs / g(x).abs
                pos_imp_eq_abs(g(x))
                g(x).is_positive
                g(x) = g(x).abs
                (g(a + delta1 * Real.one_half) / g(x)).abs < eta
                g(a + delta1 * Real.one_half).abs / g(x).abs < eta
                div_lt_mul_pos(g(a + delta1 * Real.one_half).abs, g(x).abs, eta)
                g(a + delta1 * Real.one_half).abs < eta * g(x).abs
                mul_lt_mul_of_pos_right[Real](eta, Real.1, g(x).abs)
                eta * g(x).abs < Real.1 * g(x).abs
                mul_one_left(g(x).abs)
                Real.1 * g(x).abs = g(x).abs
                eta * g(x).abs < g(x).abs
                lt_trans[Real](g(a + delta1 * Real.one_half).abs, eta * g(x).abs, g(x).abs)
                g(a + delta1 * Real.one_half).abs < g(x).abs
                abs_lt_abs_imp_ne(g(a + delta1 * Real.one_half), g(x))
                g(a + delta1 * Real.one_half) != g(x)
                // The identity f(x)/g(x) = R(x) * S(x).
                lhopital_infinity_identity(f, g, a + delta1 * Real.one_half, x)
                quotient_fn(f, g, x) = lhopital_ratio_x1(f, g, a + delta1 * Real.one_half, x) *
                    lhopital_scale_x1(f, g, a + delta1 * Real.one_half, x)
                // The ratio bound: R(x).is_close(l, eps1).
                lhopital_infinity_ratio_pointwise(f, g, df, dg, a, b,
                    a + delta1 * Real.one_half, delta1, l, eps1, x)
                lhopital_ratio_x1(f, g, a + delta1 * Real.one_half, x).is_close(l, eps1)
                // The scale bound: S(x).is_close(1, (two*two)*eta).
                lhopital_infinity_scale_pointwise(f, g, a + delta1 * Real.one_half, x, eta)
                lhopital_scale_x1(f, g, a + delta1 * Real.one_half, x).is_close(Real.1,
                    (two * two) * eta)
                // |R - l| < eps1 and |S - 1| < (two*two)*eta.
                (lhopital_ratio_x1(f, g, a + delta1 * Real.one_half, x) - l).abs < eps1
                (lhopital_scale_x1(f, g, a + delta1 * Real.one_half, x) - Real.1).abs < (two * two) * eta
                // |R| < |l| + eps1.
                triangle_ineq(lhopital_ratio_x1(f, g, a + delta1 * Real.one_half, x) - l, l)
                ((lhopital_ratio_x1(f, g, a + delta1 * Real.one_half, x) - l) + l).abs <= (lhopital_ratio_x1(f, g, a + delta1 * Real.one_half, x) - l).abs + l.abs
                (lhopital_ratio_x1(f, g, a + delta1 * Real.one_half, x) - l) + l =
                    lhopital_ratio_x1(f, g, a + delta1 * Real.one_half, x)
                lhopital_ratio_x1(f, g, a + delta1 * Real.one_half, x).abs <= (lhopital_ratio_x1(f, g, a + delta1 * Real.one_half, x) - l).abs + l.abs
                lt_add_right((lhopital_ratio_x1(f, g, a + delta1 * Real.one_half, x) - l).abs,
                    eps1, l.abs)
                l.abs + (lhopital_ratio_x1(f, g, a + delta1 * Real.one_half, x) - l).abs < l.abs + eps1
                (lhopital_ratio_x1(f, g, a + delta1 * Real.one_half, x) - l).abs + l.abs < l.abs + eps1
                lte_lt_trans(lhopital_ratio_x1(f, g, a + delta1 * Real.one_half, x).abs,
                    (lhopital_ratio_x1(f, g, a + delta1 * Real.one_half, x) - l).abs + l.abs,
                    l.abs + eps1)
                lhopital_ratio_x1(f, g, a + delta1 * Real.one_half, x).abs < l.abs + eps1
                // |R| * |S - 1| < (l.abs + eps1) * (two*two) * eta = (l.abs + eps1) * eps2.
                mul_abs(lhopital_ratio_x1(f, g, a + delta1 * Real.one_half, x),
                    lhopital_scale_x1(f, g, a + delta1 * Real.one_half, x) - Real.1)
                lhopital_ratio_x1(f, g, a + delta1 * Real.one_half, x).abs *
                    (lhopital_scale_x1(f, g, a + delta1 * Real.one_half, x) - Real.1).abs =
                    (lhopital_ratio_x1(f, g, a + delta1 * Real.one_half, x) *
                        (lhopital_scale_x1(f, g, a + delta1 * Real.one_half, x) - Real.1)).abs
                abs_gte_zero(lhopital_scale_x1(f, g, a + delta1 * Real.one_half, x) - Real.1)
                Real.0 <= (lhopital_scale_x1(f, g, a + delta1 * Real.one_half, x) - Real.1).abs
                multiply_inequality_with_nonnegative_element[Real](
                    lhopital_ratio_x1(f, g, a + delta1 * Real.one_half, x).abs,
                    l.abs + eps1,
                    (lhopital_scale_x1(f, g, a + delta1 * Real.one_half, x) - Real.1).abs)
                lhopital_ratio_x1(f, g, a + delta1 * Real.one_half, x).abs *
                    (lhopital_scale_x1(f, g, a + delta1 * Real.one_half, x) - Real.1).abs <= (l.abs + eps1) *
                    (lhopital_scale_x1(f, g, a + delta1 * Real.one_half, x) - Real.1).abs
                gt_zero_imp_pos(l.abs + eps1)
                (l.abs + eps1).is_positive
                pos_gt_zero(l.abs + eps1)
                l.abs + eps1 > Real.0
                mul_lt_mul_of_pos_right[Real](
                    (lhopital_scale_x1(f, g, a + delta1 * Real.one_half, x) - Real.1).abs,
                    (two * two) * eta, l.abs + eps1)
                (l.abs + eps1) * (lhopital_scale_x1(f, g, a + delta1 * Real.one_half, x) - Real.1).abs < (l.abs + eps1) * ((two * two) * eta)
                lte_lt_trans(
                    lhopital_ratio_x1(f, g, a + delta1 * Real.one_half, x).abs *
                        (lhopital_scale_x1(f, g, a + delta1 * Real.one_half, x) - Real.1).abs,
                    (l.abs + eps1) *
                        (lhopital_scale_x1(f, g, a + delta1 * Real.one_half, x) - Real.1).abs,
                    (l.abs + eps1) * ((two * two) * eta))
                lhopital_ratio_x1(f, g, a + delta1 * Real.one_half, x).abs *
                    (lhopital_scale_x1(f, g, a + delta1 * Real.one_half, x) - Real.1).abs < (l.abs + eps1) * ((two * two) * eta)
                // (l.abs + eps1) * (two*two) * eta = (l.abs + eps1) * eps2 < eps1.
                // (two*two)*eta = eps2
                mul_assoc(two, two, eta)
                (two * two) * eta = two * (two * eta)
                mul_assoc(eps2, Real.one_half, Real.one_half)
                (eps2 * Real.one_half) * Real.one_half = eps2 * (Real.one_half * Real.one_half)
                eta = eps2 * (Real.one_half * Real.one_half)
                mul_distrib_left(Real.one_half, Real.1, Real.1)
                (Real.1 + Real.1) * Real.one_half =
                    Real.1 * Real.one_half + Real.1 * Real.one_half
                two * Real.one_half = Real.one_half + Real.one_half
                one_half_plus_one_half
                Real.one_half + Real.one_half = Real.1
                two * Real.one_half = Real.1
                two * (two * Real.one_half) = two * Real.1
                mul_one_right(two)
                two * Real.1 = two
                two * (two * Real.one_half) = two
                eta = eps2 * (Real.one_half * Real.one_half)
                two * (two * eta) = two * (two * (eps2 * (Real.one_half * Real.one_half)))
                mul_assoc(two, two, eps2 * (Real.one_half * Real.one_half))
                (two * two) * (eps2 * (Real.one_half * Real.one_half)) =
                    two * (two * (eps2 * (Real.one_half * Real.one_half)))
                two * (two * (eps2 * (Real.one_half * Real.one_half))) =
                    (two * two) * (eps2 * (Real.one_half * Real.one_half))
                mul_assoc(two * two, eps2, Real.one_half * Real.one_half)
                ((two * two) * eps2) * (Real.one_half * Real.one_half) =
                    (two * two) * (eps2 * (Real.one_half * Real.one_half))
                (two * two) * (eps2 * (Real.one_half * Real.one_half)) =
                    ((two * two) * eps2) * (Real.one_half * Real.one_half)
                real_mul_comm(two * two, eps2)
                (two * two) * eps2 = eps2 * (two * two)
                ((two * two) * eps2) * (Real.one_half * Real.one_half) =
                    (eps2 * (two * two)) * (Real.one_half * Real.one_half)
                mul_assoc(eps2, two * two, Real.one_half * Real.one_half)
                (eps2 * (two * two)) * (Real.one_half * Real.one_half) =
                    eps2 * ((two * two) * (Real.one_half * Real.one_half))
                mul_assoc(two, two, Real.one_half * Real.one_half)
                (two * two) * (Real.one_half * Real.one_half) =
                    two * (two * (Real.one_half * Real.one_half))
                mul_assoc(two, Real.one_half, Real.one_half)
                two * (Real.one_half * Real.one_half) =
                    (two * Real.one_half) * Real.one_half
                two * (two * (Real.one_half * Real.one_half)) =
                    two * ((two * Real.one_half) * Real.one_half)
                (two * Real.one_half) * Real.one_half = Real.1 * Real.one_half
                two * ((two * Real.one_half) * Real.one_half) =
                    two * (Real.1 * Real.one_half)
                mul_one_left(Real.one_half)
                Real.1 * Real.one_half = Real.one_half
                two * (Real.1 * Real.one_half) = two * Real.one_half
                two * Real.one_half = Real.1
                two * (Real.1 * Real.one_half) = Real.1
                two * (two * (Real.one_half * Real.one_half)) = Real.1
                (two * two) * (Real.one_half * Real.one_half) = Real.1
                eps2 * ((two * two) * (Real.one_half * Real.one_half)) =
                    eps2 * Real.1
                mul_one_right(eps2)
                eps2 * Real.1 = eps2
                eps2 * ((two * two) * (Real.one_half * Real.one_half)) = eps2
                ((two * two) * eps2) * (Real.one_half * Real.one_half) = eps2
                (two * two) * (eps2 * (Real.one_half * Real.one_half)) = eps2
                two * (two * (eps2 * (Real.one_half * Real.one_half))) = eps2
                two * (two * eta) = eps2
                (two * two) * eta = eps2
                (l.abs + eps1) * ((two * two) * eta) = (l.abs + eps1) * eps2
                lt_add_pos(l.abs + eps1, Real.1)
                l.abs + eps1 < (l.abs + eps1) + Real.1
                (l.abs + eps1) + Real.1 = l.abs + eps1 + Real.1
                l.abs + eps1 < l.abs + eps1 + Real.1
                mul_lt_mul_of_pos_right[Real](l.abs + eps1, l.abs + eps1 + Real.1, eps1)
                (l.abs + eps1) * eps1 < (l.abs + eps1 + Real.1) * eps1
                lt_imp_ne(Real.0, l.abs + eps1 + Real.1)
                Real.0 != l.abs + eps1 + Real.1
                l.abs + eps1 + Real.1 != Real.0
                div_lt_div_pos((l.abs + eps1) * eps1, (l.abs + eps1 + Real.1) * eps1,
                    l.abs + eps1 + Real.1)
                ((l.abs + eps1) * eps1) / (l.abs + eps1 + Real.1) < ((l.abs + eps1 + Real.1) * eps1) / (l.abs + eps1 + Real.1)
                div_mul_cancel_left(l.abs + eps1 + Real.1, eps1)
                ((l.abs + eps1 + Real.1) * eps1) / (l.abs + eps1 + Real.1) = eps1
                ((l.abs + eps1) * eps1) / (l.abs + eps1 + Real.1) < eps1
                mul_assoc(l.abs + eps1, eps1, Real.1)
                (l.abs + eps1) * (eps1 / (l.abs + eps1 + Real.1)) =
                    ((l.abs + eps1) * eps1) / (l.abs + eps1 + Real.1)
                eps2 = eps1 / (l.abs + eps1 + Real.1)
                (l.abs + eps1) * eps2 = ((l.abs + eps1) * eps1) / (l.abs + eps1 + Real.1)
                (l.abs + eps1) * eps2 < eps1
                (l.abs + eps1) * ((two * two) * eta) < eps1
                lt_trans[Real](
                    lhopital_ratio_x1(f, g, a + delta1 * Real.one_half, x).abs *
                        (lhopital_scale_x1(f, g, a + delta1 * Real.one_half, x) - Real.1).abs,
                    (l.abs + eps1) * ((two * two) * eta), eps1)
                lhopital_ratio_x1(f, g, a + delta1 * Real.one_half, x).abs *
                    (lhopital_scale_x1(f, g, a + delta1 * Real.one_half, x) - Real.1).abs < eps1
                // |R*S - l| <= |R*(S-1)| + |R - l| < eps1 + eps1 < eps.
                triangle_ineq(
                    lhopital_ratio_x1(f, g, a + delta1 * Real.one_half, x) *
                        (lhopital_scale_x1(f, g, a + delta1 * Real.one_half, x) - Real.1),
                    lhopital_ratio_x1(f, g, a + delta1 * Real.one_half, x) - l)
                (lhopital_ratio_x1(f, g, a + delta1 * Real.one_half, x) *
                        (lhopital_scale_x1(f, g, a + delta1 * Real.one_half, x) - Real.1) +
                    (lhopital_ratio_x1(f, g, a + delta1 * Real.one_half, x) - l)).abs <= (lhopital_ratio_x1(f, g, a + delta1 * Real.one_half, x) *
                        (lhopital_scale_x1(f, g, a + delta1 * Real.one_half, x) - Real.1)).abs +
                    (lhopital_ratio_x1(f, g, a + delta1 * Real.one_half, x) - l).abs
                mul_sub_distrib_right(lhopital_ratio_x1(f, g, a + delta1 * Real.one_half, x),
                    lhopital_scale_x1(f, g, a + delta1 * Real.one_half, x), Real.1)
                lhopital_ratio_x1(f, g, a + delta1 * Real.one_half, x) *
                    (lhopital_scale_x1(f, g, a + delta1 * Real.one_half, x) - Real.1) =
                    lhopital_ratio_x1(f, g, a + delta1 * Real.one_half, x) *
                        lhopital_scale_x1(f, g, a + delta1 * Real.one_half, x) -
                    lhopital_ratio_x1(f, g, a + delta1 * Real.one_half, x)
                lhopital_ratio_x1(f, g, a + delta1 * Real.one_half, x) *
                        (lhopital_scale_x1(f, g, a + delta1 * Real.one_half, x) - Real.1) +
                    (lhopital_ratio_x1(f, g, a + delta1 * Real.one_half, x) - l) =
                    lhopital_ratio_x1(f, g, a + delta1 * Real.one_half, x) *
                        lhopital_scale_x1(f, g, a + delta1 * Real.one_half, x) - l
                (lhopital_ratio_x1(f, g, a + delta1 * Real.one_half, x) *
                        lhopital_scale_x1(f, g, a + delta1 * Real.one_half, x) - l).abs <= (lhopital_ratio_x1(f, g, a + delta1 * Real.one_half, x) *
                        (lhopital_scale_x1(f, g, a + delta1 * Real.one_half, x) - Real.1)).abs +
                    (lhopital_ratio_x1(f, g, a + delta1 * Real.one_half, x) - l).abs
                add_lt_lt(
                    (lhopital_ratio_x1(f, g, a + delta1 * Real.one_half, x) *
                        (lhopital_scale_x1(f, g, a + delta1 * Real.one_half, x) - Real.1)).abs,
                    eps1,
                    (lhopital_ratio_x1(f, g, a + delta1 * Real.one_half, x) - l).abs,
                    eps1)
                (lhopital_ratio_x1(f, g, a + delta1 * Real.one_half, x) *
                        (lhopital_scale_x1(f, g, a + delta1 * Real.one_half, x) - Real.1)).abs +
                    (lhopital_ratio_x1(f, g, a + delta1 * Real.one_half, x) - l).abs < eps1 + eps1
                lte_lt_trans(
                    (lhopital_ratio_x1(f, g, a + delta1 * Real.one_half, x) *
                        lhopital_scale_x1(f, g, a + delta1 * Real.one_half, x) - l).abs,
                    (lhopital_ratio_x1(f, g, a + delta1 * Real.one_half, x) *
                        (lhopital_scale_x1(f, g, a + delta1 * Real.one_half, x) - Real.1)).abs +
                        (lhopital_ratio_x1(f, g, a + delta1 * Real.one_half, x) - l).abs,
                    eps1 + eps1)
                (lhopital_ratio_x1(f, g, a + delta1 * Real.one_half, x) *
                    lhopital_scale_x1(f, g, a + delta1 * Real.one_half, x) - l).abs < eps1 + eps1
                lt_add_pos(eps1 + eps1, eps1)
                eps1 + eps1 < (eps1 + eps1) + eps1
                (eps1 + eps1) + eps1 = eps1 + eps1 + eps1
                eps1 + eps1 < eps1 + eps1 + eps1
                lt_trans[Real](eps1 + eps1, eps1 + eps1 + eps1, eps)
                eps1 + eps1 < eps
                lt_trans[Real](
                    (lhopital_ratio_x1(f, g, a + delta1 * Real.one_half, x) *
                        lhopital_scale_x1(f, g, a + delta1 * Real.one_half, x) - l).abs,
                    eps1 + eps1, eps)
                (lhopital_ratio_x1(f, g, a + delta1 * Real.one_half, x) *
                    lhopital_scale_x1(f, g, a + delta1 * Real.one_half, x) - l).abs < eps
                (lhopital_ratio_x1(f, g, a + delta1 * Real.one_half, x) *
                    lhopital_scale_x1(f, g, a + delta1 * Real.one_half, x)).is_close(l, eps)
                quotient_fn(f, g, x) = lhopital_ratio_x1(f, g, a + delta1 * Real.one_half, x) *
                    lhopital_scale_x1(f, g, a + delta1 * Real.one_half, x)
                quotient_fn(f, g, x).is_close(l, eps)
            }
        }
        delta.is_positive and a + delta < b and
        forall(x: Real) {
            a < x and x < a + delta implies quotient_fn(f, g, x).is_close(l, eps)
        }
        exists(delta3: Real) {
            delta3.is_positive and a + delta3 < b and
            forall(x: Real) {
                a < x and x < a + delta3 implies quotient_fn(f, g, x).is_close(l, eps)
            }
        }
    }
}

/// L'Hôpital's rule for the ∞/∞ indeterminate form, in ε-δ form: if f and g
/// are differentiable on (a, b), g' stays nonzero there, f'/g' has right
/// limit l at a, and f and g tend to +∞ as x -> a+, then for every tolerance
/// eps there is a punctured right neighborhood of a on which f/g stays within
/// eps of l.  The predicate `limit_at_right` unfolds to exactly this
/// statement, so this is the right-limit form of the rule.
theorem lhopital_infinity_form(f: Real -> Real, g: Real -> Real, df: Real -> Real,
    dg: Real -> Real, a: Real, b: Real, l: Real) {
    continuous(f) and continuous(g) and is_derivative_fn(f, df) and
    is_derivative_fn(g, dg) and
    (forall(x: Real) { a < x and x < b implies dg(x) != Real.0 }) and
    limit_at_right(quotient_fn(df, dg), a, b, l) and
    tends_infinity_right(f, a, b) and tends_infinity_right(g, a, b)
    implies forall(eps: Real) {
        eps.is_positive implies exists(delta: Real) {
            delta.is_positive and a + delta < b and
            forall(x: Real) {
                a < x and x < a + delta implies quotient_fn(f, g, x).is_close(l, eps)
            }
        }
    }
} by {
    if continuous(f) and continuous(g) and is_derivative_fn(f, df) and
       is_derivative_fn(g, dg) and
       (forall(x: Real) { a < x and x < b implies dg(x) != Real.0 }) and
       limit_at_right(quotient_fn(df, dg), a, b, l) and
       tends_infinity_right(f, a, b) and tends_infinity_right(g, a, b) {
        forall(eps: Real) {
            if eps.is_positive {
                lhopital_infinity_core(f, g, df, dg, a, b, l, eps)
                exists(delta: Real) {
                    delta.is_positive and a + delta < b and
                    forall(x: Real) {
                        a < x and x < a + delta implies quotient_fn(f, g, x).is_close(l, eps)
                    }
                }
            }
        }
    }
}

/// The sequential form of the ∞/∞ rule: every sequence approaching a from
/// above (with terms inside (a, b)) is carried by f/g to a sequence
/// converging to l.  This matches the style of the 0/0 rule in
/// real/lhopital.ac.  The conclusion is the ε-δ form of sequential
/// convergence: `converges_to` unfolds to exactly this statement, and the
/// proof of the tail bounds is included here so that the theorem stands on
/// its own.
theorem lhopital_infinity_seq_limit(f: Real -> Real, g: Real -> Real,
    df: Real -> Real, dg: Real -> Real, a: Real, b: Real, l: Real,
    q: Nat -> Real) {
    continuous(f) and continuous(g) and is_derivative_fn(f, df) and
    is_derivative_fn(g, dg) and
    (forall(x: Real) { a < x and x < b implies dg(x) != Real.0 }) and
    (forall(n: Nat) { a < q(n) and q(n) < b }) and converges_to(q, a) and
    limit_at_right(quotient_fn(df, dg), a, b, l) and
    tends_infinity_right(f, a, b) and tends_infinity_right(g, a, b)
    implies forall(eps: Real) {
        eps.is_positive implies exists(n: Nat) {
            tail_bound(compose(quotient_fn(f, g), q), l, n, eps)
        }
    }
} by {
    if continuous(f) and continuous(g) and is_derivative_fn(f, df) and
       is_derivative_fn(g, dg) and
       (forall(x: Real) { a < x and x < b implies dg(x) != Real.0 }) and
       (forall(n: Nat) { a < q(n) and q(n) < b }) and converges_to(q, a) and
       limit_at_right(quotient_fn(df, dg), a, b, l) and
       tends_infinity_right(f, a, b) and tends_infinity_right(g, a, b) {
        converges_to(q, a) = forall(eps2: Real) {
            eps2.is_positive implies exists(n2: Nat) {
                tail_bound(q, a, n2, eps2)
            }
        }
        forall(eps: Real) {
            if eps.is_positive {
                lhopital_infinity_core(f, g, df, dg, a, b, l, eps)
                let delta: Real satisfy {
                    delta.is_positive and a + delta < b and
                    forall(x: Real) {
                        a < x and x < a + delta implies quotient_fn(f, g, x).is_close(l, eps)
                    }
                }
                let n0: Nat satisfy {
                    tail_bound(q, a, n0, delta)
                }
                forall(i: Nat) {
                    if n0 <= i {
                        tail_bound_implies_is_close(q, a, n0, delta, i)
                        q(i).is_close(a, delta)
                        close_imp_bounds(q(i), a, delta)
                        q(i) < a + delta
                        forall(n: Nat) { a < q(n) and q(n) < b }
                        a < q(i)
                        a < q(i) and q(i) < a + delta
                        forall(x: Real) {
                            a < x and x < a + delta implies quotient_fn(f, g, x).is_close(l, eps)
                        }
                        quotient_fn(f, g, q(i)).is_close(l, eps)
                        compose(quotient_fn(f, g), q, i) = quotient_fn(f, g, q(i))
                        compose(quotient_fn(f, g), q, i).is_close(l, eps)
                    }
                }
                tail_bound(compose(quotient_fn(f, g), q), l, n0, eps)
                exists(n: Nat) {
                    tail_bound(compose(quotient_fn(f, g), q), l, n, eps)
                }
            }
        }
    }
}
