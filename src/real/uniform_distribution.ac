/// The continuous uniform distribution on a closed interval.
///
/// The probability density function of the uniform law on [a, b] is the
/// constant 1 / (b - a) on the interval and zero outside it.  On [0, 1] the
/// density is the constant one.  This file develops:
///   - the density functions and their nonnegativity and boundedness,
///   - the total mass: the integral of the density over its interval is one,
///   - the expectation E[X] = (a + b) / 2, computed as the integral of
///     x * f(x),
///   - the variance Var(X) = (b - a)^2 / 12, computed as the integral of
///     (x - E[X])^2 * f(x),
///   - the cumulative distribution function F(x) = (x - a) / (b - a) on
///     [a, b], defined as the integral of the density from a to x,
///   - subinterval probabilities: the mass of [u, v] inside [a, b] is
///     (v - u) / (b - a).
///
/// All values are Riemann integrals over finite intervals.  The density is
/// exactly constant on the closed interval of integration, so each integral
/// of a piecewise density agrees with the integral of a constant function;
/// the bridge is real.integral_and_integrable_eq_on, and the constant
/// integrals come from the fundamental theorem of calculus.

from real.real_field import Real
from real.integral import integral, is_integrable, interval_contains, interval_contains_left,
    interval_contains_right
from real.real_base import neg_distrib, one_half_plus_one_half
from real.real_ring import real_mul_comm, square_nonneg
from real.integral_exp import ftc2_general
from real.calculus_api import derivative_fn_identity, is_derivative_fn
from real.continuity_base import continuous
from real.continuity_sequences import identity_function_is_continuous
from real.integrability import interior_constant_integrable
from real.integral_polynomial_values import integral_identity, centered_square_unit,
    integral_centered_square_unit_third, integral_and_integrable_eq_on
from data.basic.functions import identity_fn
from data.basic.logic import eq_true_intro
from order import lt_imp_lte, lte_trans
from ordered_field import zero_is_smaller_than_one, mul_le_mul_of_nonneg_right
from algebra.add_ordered_group import add_le_add_right

numerals Real

/// The probability density of the uniform law on [0, 1]: one on the closed
/// unit interval and zero elsewhere.
define uniform_pdf(x: Real) -> Real {
    if interval_contains(Real.0, Real.1, x) { Real.1 } else { Real.0 }
}

/// The uniform density is one on [0, 1].
theorem uniform_pdf_one_on_unit(x: Real) {
    interval_contains(Real.0, Real.1, x) implies uniform_pdf(x) = Real.1
} by {
    if interval_contains(Real.0, Real.1, x) {
        uniform_pdf(x) = Real.1
    }
}

/// The uniform density vanishes outside [0, 1].
theorem uniform_pdf_zero_outside_unit(x: Real) {
    not interval_contains(Real.0, Real.1, x) implies uniform_pdf(x) = Real.0
} by {
    if not interval_contains(Real.0, Real.1, x) {
        uniform_pdf(x) = Real.0
    }
}

/// The uniform density is nonnegative.
theorem uniform_pdf_nonneg(x: Real) {
    Real.0 <= uniform_pdf(x)
} by {
    if interval_contains(Real.0, Real.1, x) {
        uniform_pdf_one_on_unit(x)
        uniform_pdf(x) = Real.1
        zero_is_smaller_than_one[Real]
        Real.0 < Real.1
        lt_imp_lte(Real.0, Real.1)
        Real.0 <= Real.1
        Real.0 <= uniform_pdf(x)
    }
    if not interval_contains(Real.0, Real.1, x) {
        uniform_pdf_zero_outside_unit(x)
        uniform_pdf(x) = Real.0
        Real.0 <= uniform_pdf(x)
    }
    interval_contains(Real.0, Real.1, x) or not interval_contains(Real.0, Real.1, x)
    Real.0 <= uniform_pdf(x)
}

/// The uniform density is bounded above by one.
theorem uniform_pdf_le_one(x: Real) {
    uniform_pdf(x) <= Real.1
} by {
    if interval_contains(Real.0, Real.1, x) {
        uniform_pdf_one_on_unit(x)
        uniform_pdf(x) = Real.1
        Real.1 <= Real.1
        uniform_pdf(x) <= Real.1
    }
    if not interval_contains(Real.0, Real.1, x) {
        uniform_pdf_zero_outside_unit(x)
        uniform_pdf(x) = Real.0
        zero_is_smaller_than_one[Real]
        Real.0 < Real.1
        lt_imp_lte(Real.0, Real.1)
        Real.0 <= Real.1
        uniform_pdf(x) <= Real.1
    }
    interval_contains(Real.0, Real.1, x) or not interval_contains(Real.0, Real.1, x)
    uniform_pdf(x) <= Real.1
}

/// On the open unit interval the density is constantly one.
lemma uniform_pdf_interior_const_unit(t: Real) {
    Real.0 < t and t < Real.1 implies uniform_pdf(t) = Real.1
} by {
    if Real.0 < t and t < Real.1 {
        lt_imp_lte(Real.0, t)
        Real.0 <= t
        lt_imp_lte(t, Real.1)
        t <= Real.1
        interval_contains(Real.0, Real.1, t)
        uniform_pdf_one_on_unit(t)
        uniform_pdf(t) = Real.1
    }
}

/// On [0, 1] the density agrees with the constant one function.
lemma uniform_pdf_eq_const_one_on_unit(t: Real) {
    interval_contains(Real.0, Real.1, t) implies uniform_pdf(t) = constant[Real, Real](Real.1, t)
} by {
    if interval_contains(Real.0, Real.1, t) {
        uniform_pdf_one_on_unit(t)
        uniform_pdf(t) = Real.1
        constant[Real, Real](Real.1, t) = Real.1
        uniform_pdf(t) = constant[Real, Real](Real.1, t)
    }
}

/// On [0, 1] the density is bounded between zero and one.
lemma uniform_pdf_bounds_unit(t: Real) {
    interval_contains(Real.0, Real.1, t) implies (Real.0 <= uniform_pdf(t) and uniform_pdf(t) <= Real.1)
} by {
    if interval_contains(Real.0, Real.1, t) {
        uniform_pdf_nonneg(t)
        Real.0 <= uniform_pdf(t)
        uniform_pdf_le_one(t)
        uniform_pdf(t) <= Real.1
        Real.0 <= uniform_pdf(t) and uniform_pdf(t) <= Real.1
    }
}

/// The constant one function is integrable on [0, 1].
lemma const_one_integrable_unit {
    is_integrable(constant[Real, Real](Real.1), Real.0, Real.1)
} by {
    zero_is_smaller_than_one[Real]
    Real.0 < Real.1
    lt_imp_lte(Real.0, Real.1)
    Real.0 <= Real.1
    forall(t: Real) {
        if Real.0 < t and t < Real.1 {
            constant[Real, Real](Real.1, t) = Real.1
        }
    }
    forall(t: Real) { Real.0 < t and t < Real.1 implies constant[Real, Real](Real.1, t) = Real.1 }
    forall(t: Real) {
        if interval_contains(Real.0, Real.1, t) {
            constant[Real, Real](Real.1, t) = Real.1
            Real.0 <= constant[Real, Real](Real.1, t)
        }
    }
    forall(t: Real) { interval_contains(Real.0, Real.1, t) implies Real.0 <= constant[Real, Real](Real.1, t) }
    forall(t: Real) {
        if interval_contains(Real.0, Real.1, t) {
            constant[Real, Real](Real.1, t) = Real.1
            constant[Real, Real](Real.1, t) <= Real.1
        }
    }
    forall(t: Real) { interval_contains(Real.0, Real.1, t) implies constant[Real, Real](Real.1, t) <= Real.1 }
    eq_true_intro(forall(t: Real) { Real.0 < t and t < Real.1 implies constant[Real, Real](Real.1, t) = Real.1 })
    (forall(t: Real) { Real.0 < t and t < Real.1 implies constant[Real, Real](Real.1, t) = Real.1 }) = true
    eq_true_intro(forall(t: Real) { interval_contains(Real.0, Real.1, t) implies Real.0 <= constant[Real, Real](Real.1, t) })
    (forall(t: Real) { interval_contains(Real.0, Real.1, t) implies Real.0 <= constant[Real, Real](Real.1, t) }) = true
    eq_true_intro(forall(t: Real) { interval_contains(Real.0, Real.1, t) implies constant[Real, Real](Real.1, t) <= Real.1 })
    (forall(t: Real) { interval_contains(Real.0, Real.1, t) implies constant[Real, Real](Real.1, t) <= Real.1 }) = true
    Real.0 <= Real.1 and
        (forall(t: Real) { Real.0 < t and t < Real.1 implies constant[Real, Real](Real.1, t) = Real.1 }) and
        (forall(t: Real) { interval_contains(Real.0, Real.1, t) implies Real.0 <= constant[Real, Real](Real.1, t) }) and
        (forall(t: Real) { interval_contains(Real.0, Real.1, t) implies constant[Real, Real](Real.1, t) <= Real.1 })
    interior_constant_integrable(constant[Real, Real](Real.1), Real.0, Real.1, Real.1, Real.0, Real.1)
    is_integrable(constant[Real, Real](Real.1), Real.0, Real.1)
}

/// The integral of the constant one function over [0, 1] is one.
lemma const_one_integral_unit {
    integral(constant[Real, Real](Real.1), Real.0, Real.1) = Real.1
} by {
    zero_is_smaller_than_one[Real]
    Real.0 < Real.1
    lt_imp_lte(Real.0, Real.1)
    Real.0 <= Real.1
    const_one_integrable_unit
    is_integrable(constant[Real, Real](Real.1), Real.0, Real.1)
    identity_function_is_continuous
    continuous(identity_fn[Real])
    derivative_fn_identity
    is_derivative_fn(identity_fn[Real], constant[Real, Real](Real.1))
    forall(t: Real) {
        if interval_contains(Real.0, Real.1, t) {
            constant[Real, Real](Real.1, t) = Real.1
            Real.0 <= constant[Real, Real](Real.1, t)
        }
    }
    forall(t: Real) { interval_contains(Real.0, Real.1, t) implies Real.0 <= constant[Real, Real](Real.1, t) }
    forall(t: Real) {
        if interval_contains(Real.0, Real.1, t) {
            constant[Real, Real](Real.1, t) = Real.1
            constant[Real, Real](Real.1, t) <= Real.1
        }
    }
    forall(t: Real) { interval_contains(Real.0, Real.1, t) implies constant[Real, Real](Real.1, t) <= Real.1 }
    eq_true_intro(continuous(identity_fn[Real]))
    (continuous(identity_fn[Real])) = true
    eq_true_intro(is_derivative_fn(identity_fn[Real], constant[Real, Real](Real.1)))
    (is_derivative_fn(identity_fn[Real], constant[Real, Real](Real.1))) = true
    eq_true_intro(forall(t: Real) { interval_contains(Real.0, Real.1, t) implies Real.0 <= constant[Real, Real](Real.1, t) })
    (forall(t: Real) { interval_contains(Real.0, Real.1, t) implies Real.0 <= constant[Real, Real](Real.1, t) }) = true
    eq_true_intro(forall(t: Real) { interval_contains(Real.0, Real.1, t) implies constant[Real, Real](Real.1, t) <= Real.1 })
    (forall(t: Real) { interval_contains(Real.0, Real.1, t) implies constant[Real, Real](Real.1, t) <= Real.1 }) = true
    Real.0 <= Real.1 and continuous(identity_fn[Real]) and
        is_derivative_fn(identity_fn[Real], constant[Real, Real](Real.1)) and
        is_integrable(constant[Real, Real](Real.1), Real.0, Real.1) and
        (forall(t: Real) { interval_contains(Real.0, Real.1, t) implies Real.0 <= constant[Real, Real](Real.1, t) }) and
        (forall(t: Real) { interval_contains(Real.0, Real.1, t) implies constant[Real, Real](Real.1, t) <= Real.1 })
    ftc2_general(constant[Real, Real](Real.1), identity_fn[Real], Real.0, Real.1, Real.0, Real.1)
    integral(constant[Real, Real](Real.1), Real.0, Real.1) = identity_fn[Real](Real.1) - identity_fn[Real](Real.0)
    identity_fn[Real](Real.1) = Real.1
    identity_fn[Real](Real.0) = Real.0
    identity_fn[Real](Real.1) - identity_fn[Real](Real.0) = Real.1 - Real.0
    Real.1 - Real.0 = Real.1
    integral(constant[Real, Real](Real.1), Real.0, Real.1) = Real.1
}

/// The uniform density is integrable on [0, 1].
theorem uniform_pdf_integrable_unit {
    is_integrable(uniform_pdf, Real.0, Real.1)
} by {
    zero_is_smaller_than_one[Real]
    Real.0 < Real.1
    lt_imp_lte(Real.0, Real.1)
    Real.0 <= Real.1
    forall(t: Real) {
        uniform_pdf_interior_const_unit(t)
    }
    forall(t: Real) { Real.0 < t and t < Real.1 implies uniform_pdf(t) = Real.1 }
    forall(t: Real) {
        uniform_pdf_bounds_unit(t)
        interval_contains(Real.0, Real.1, t) implies (Real.0 <= uniform_pdf(t) and uniform_pdf(t) <= Real.1)
        interval_contains(Real.0, Real.1, t) implies Real.0 <= uniform_pdf(t)
    }
    forall(t: Real) { interval_contains(Real.0, Real.1, t) implies Real.0 <= uniform_pdf(t) }
    forall(t: Real) {
        uniform_pdf_bounds_unit(t)
        interval_contains(Real.0, Real.1, t) implies (Real.0 <= uniform_pdf(t) and uniform_pdf(t) <= Real.1)
        interval_contains(Real.0, Real.1, t) implies uniform_pdf(t) <= Real.1
    }
    forall(t: Real) { interval_contains(Real.0, Real.1, t) implies uniform_pdf(t) <= Real.1 }
    eq_true_intro(forall(t: Real) { Real.0 < t and t < Real.1 implies uniform_pdf(t) = Real.1 })
    (forall(t: Real) { Real.0 < t and t < Real.1 implies uniform_pdf(t) = Real.1 }) = true
    eq_true_intro(forall(t: Real) { interval_contains(Real.0, Real.1, t) implies Real.0 <= uniform_pdf(t) })
    (forall(t: Real) { interval_contains(Real.0, Real.1, t) implies Real.0 <= uniform_pdf(t) }) = true
    eq_true_intro(forall(t: Real) { interval_contains(Real.0, Real.1, t) implies uniform_pdf(t) <= Real.1 })
    (forall(t: Real) { interval_contains(Real.0, Real.1, t) implies uniform_pdf(t) <= Real.1 }) = true
    Real.0 <= Real.1 and
        (forall(t: Real) { Real.0 < t and t < Real.1 implies uniform_pdf(t) = Real.1 }) and
        (forall(t: Real) { interval_contains(Real.0, Real.1, t) implies Real.0 <= uniform_pdf(t) }) and
        (forall(t: Real) { interval_contains(Real.0, Real.1, t) implies uniform_pdf(t) <= Real.1 })
    interior_constant_integrable(uniform_pdf, Real.0, Real.1, Real.1, Real.0, Real.1)
    is_integrable(uniform_pdf, Real.0, Real.1)
}

/// The total mass of the uniform density on [0, 1] is one.
theorem uniform_pdf_integral_unit {
    integral(uniform_pdf, Real.0, Real.1) = Real.1
} by {
    zero_is_smaller_than_one[Real]
    Real.0 < Real.1
    lt_imp_lte(Real.0, Real.1)
    Real.0 <= Real.1
    const_one_integrable_unit
    is_integrable(constant[Real, Real](Real.1), Real.0, Real.1)
    forall(t: Real) {
        uniform_pdf_eq_const_one_on_unit(t)
        interval_contains(Real.0, Real.1, t) implies uniform_pdf(t) = constant[Real, Real](Real.1, t)
    }
    forall(t: Real) { interval_contains(Real.0, Real.1, t) implies uniform_pdf(t) = constant[Real, Real](Real.1, t) }
    forall(t: Real) {
        uniform_pdf_bounds_unit(t)
        interval_contains(Real.0, Real.1, t) implies (Real.0 <= uniform_pdf(t) and uniform_pdf(t) <= Real.1)
        interval_contains(Real.0, Real.1, t) implies Real.0 <= uniform_pdf(t)
    }
    forall(t: Real) { interval_contains(Real.0, Real.1, t) implies Real.0 <= uniform_pdf(t) }
    forall(t: Real) {
        uniform_pdf_bounds_unit(t)
        interval_contains(Real.0, Real.1, t) implies (Real.0 <= uniform_pdf(t) and uniform_pdf(t) <= Real.1)
        interval_contains(Real.0, Real.1, t) implies uniform_pdf(t) <= Real.1
    }
    forall(t: Real) { interval_contains(Real.0, Real.1, t) implies uniform_pdf(t) <= Real.1 }
    eq_true_intro(forall(t: Real) { interval_contains(Real.0, Real.1, t) implies uniform_pdf(t) = constant[Real, Real](Real.1, t) })
    (forall(t: Real) { interval_contains(Real.0, Real.1, t) implies uniform_pdf(t) = constant[Real, Real](Real.1, t) }) = true
    eq_true_intro(forall(t: Real) { interval_contains(Real.0, Real.1, t) implies Real.0 <= uniform_pdf(t) })
    (forall(t: Real) { interval_contains(Real.0, Real.1, t) implies Real.0 <= uniform_pdf(t) }) = true
    eq_true_intro(forall(t: Real) { interval_contains(Real.0, Real.1, t) implies uniform_pdf(t) <= Real.1 })
    (forall(t: Real) { interval_contains(Real.0, Real.1, t) implies uniform_pdf(t) <= Real.1 }) = true
    Real.0 <= Real.1 and is_integrable(constant[Real, Real](Real.1), Real.0, Real.1) and
        (forall(t: Real) { interval_contains(Real.0, Real.1, t) implies uniform_pdf(t) = constant[Real, Real](Real.1, t) }) and
        (forall(t: Real) { interval_contains(Real.0, Real.1, t) implies Real.0 <= uniform_pdf(t) }) and
        (forall(t: Real) { interval_contains(Real.0, Real.1, t) implies uniform_pdf(t) <= Real.1 })
    integral_and_integrable_eq_on(uniform_pdf, constant[Real, Real](Real.1), Real.0, Real.1, Real.0, Real.1)
    is_integrable(uniform_pdf, Real.0, Real.1) and
        integral(uniform_pdf, Real.0, Real.1) = integral(constant[Real, Real](Real.1), Real.0, Real.1)
    integral(uniform_pdf, Real.0, Real.1) = integral(constant[Real, Real](Real.1), Real.0, Real.1)
    const_one_integral_unit
    integral(constant[Real, Real](Real.1), Real.0, Real.1) = Real.1
    integral(uniform_pdf, Real.0, Real.1) = Real.1
}

/// The mass-weighted value of a random variable under the uniform density.
define uniform_mass_weighted(rv: Real -> Real, x: Real) -> Real {
    rv(x) * uniform_pdf(x)
}

/// The expectation of a random variable under the uniform law on [0, 1].
define uniform_expectation(rv: Real -> Real) -> Real {
    integral(uniform_mass_weighted(rv), Real.0, Real.1)
}

/// The expectation integrand of the identity agrees with the identity on [0, 1].
lemma uniform_mean_integrand_eq_on_unit(t: Real) {
    interval_contains(Real.0, Real.1, t) implies uniform_mass_weighted(identity_fn[Real], t) = identity_fn[Real](t)
} by {
    if interval_contains(Real.0, Real.1, t) {
        uniform_pdf_one_on_unit(t)
        uniform_pdf(t) = Real.1
        identity_fn[Real](t) = t
        uniform_mass_weighted(identity_fn[Real], t) = identity_fn[Real](t) * uniform_pdf(t)
        uniform_mass_weighted(identity_fn[Real], t) = identity_fn[Real](t) * Real.1
        identity_fn[Real](t) * Real.1 = identity_fn[Real](t)
        uniform_mass_weighted(identity_fn[Real], t) = identity_fn[Real](t)
    }
}

/// The mean integrand is bounded on [0, 1].
lemma uniform_mean_integrand_bounds_unit(t: Real) {
    interval_contains(Real.0, Real.1, t) implies (Real.0 <= uniform_mass_weighted(identity_fn[Real], t) and uniform_mass_weighted(identity_fn[Real], t) <= Real.1)
} by {
    if interval_contains(Real.0, Real.1, t) {
        uniform_pdf_bounds_unit(t)
        interval_contains(Real.0, Real.1, t) implies (Real.0 <= uniform_pdf(t) and uniform_pdf(t) <= Real.1)
        Real.0 <= uniform_pdf(t) and uniform_pdf(t) <= Real.1
        Real.0 <= uniform_pdf(t)
        uniform_pdf(t) <= Real.1
        interval_contains_left(Real.0, Real.1, t)
        Real.0 <= t
        identity_fn[Real](t) = t
        Real.0 <= identity_fn[Real](t)
        uniform_mass_weighted(identity_fn[Real], t) = identity_fn[Real](t) * uniform_pdf(t)
        mul_le_mul_of_nonneg_right(identity_fn[Real](t), uniform_pdf(t), Real.0)
        identity_fn[Real](t) * Real.0 <= identity_fn[Real](t) * uniform_pdf(t)
        identity_fn[Real](t) * Real.0 = Real.0
        Real.0 <= uniform_mass_weighted(identity_fn[Real], t)
        mul_le_mul_of_nonneg_right(uniform_pdf(t), Real.1, identity_fn[Real](t))
        uniform_pdf(t) * identity_fn[Real](t) <= Real.1 * identity_fn[Real](t)
        identity_fn[Real](t) * uniform_pdf(t) <= Real.1 * identity_fn[Real](t)
        Real.1 * identity_fn[Real](t) = identity_fn[Real](t)
        identity_fn[Real](t) * uniform_pdf(t) <= identity_fn[Real](t)
        uniform_mass_weighted(identity_fn[Real], t) <= identity_fn[Real](t)
        interval_contains_right(Real.0, Real.1, t)
        t <= Real.1
        identity_fn[Real](t) = t
        identity_fn[Real](t) <= Real.1
        lte_trans(uniform_mass_weighted(identity_fn[Real], t), identity_fn[Real](t), Real.1)
        uniform_mass_weighted(identity_fn[Real], t) <= Real.1
        Real.0 <= uniform_mass_weighted(identity_fn[Real], t) and uniform_mass_weighted(identity_fn[Real], t) <= Real.1
    }
}

/// The expectation of X under the uniform law on [0, 1] is one half.
theorem uniform_mean_unit {
    uniform_expectation(identity_fn[Real]) = Real.one_half
} by {
    zero_is_smaller_than_one[Real]
    Real.0 < Real.1
    lt_imp_lte(Real.0, Real.1)
    Real.0 <= Real.1
    integral_identity(Real.0, Real.1)
    is_integrable(identity_fn[Real], Real.0, Real.1) and
        integral(identity_fn[Real], Real.0, Real.1) = (Real.1 * Real.1 - Real.0 * Real.0) * Real.one_half
    is_integrable(identity_fn[Real], Real.0, Real.1)
    forall(t: Real) {
        uniform_mean_integrand_eq_on_unit(t)
        interval_contains(Real.0, Real.1, t) implies uniform_mass_weighted(identity_fn[Real], t) = identity_fn[Real](t)
    }
    forall(t: Real) { interval_contains(Real.0, Real.1, t) implies uniform_mass_weighted(identity_fn[Real], t) = identity_fn[Real](t) }
    forall(t: Real) {
        uniform_mean_integrand_bounds_unit(t)
        interval_contains(Real.0, Real.1, t) implies (Real.0 <= uniform_mass_weighted(identity_fn[Real], t) and uniform_mass_weighted(identity_fn[Real], t) <= Real.1)
        interval_contains(Real.0, Real.1, t) implies Real.0 <= uniform_mass_weighted(identity_fn[Real], t)
    }
    forall(t: Real) { interval_contains(Real.0, Real.1, t) implies Real.0 <= uniform_mass_weighted(identity_fn[Real], t) }
    forall(t: Real) {
        uniform_mean_integrand_bounds_unit(t)
        interval_contains(Real.0, Real.1, t) implies (Real.0 <= uniform_mass_weighted(identity_fn[Real], t) and uniform_mass_weighted(identity_fn[Real], t) <= Real.1)
        interval_contains(Real.0, Real.1, t) implies uniform_mass_weighted(identity_fn[Real], t) <= Real.1
    }
    forall(t: Real) { interval_contains(Real.0, Real.1, t) implies uniform_mass_weighted(identity_fn[Real], t) <= Real.1 }
    eq_true_intro(forall(t: Real) { interval_contains(Real.0, Real.1, t) implies uniform_mass_weighted(identity_fn[Real], t) = identity_fn[Real](t) })
    (forall(t: Real) { interval_contains(Real.0, Real.1, t) implies uniform_mass_weighted(identity_fn[Real], t) = identity_fn[Real](t) }) = true
    eq_true_intro(forall(t: Real) { interval_contains(Real.0, Real.1, t) implies Real.0 <= uniform_mass_weighted(identity_fn[Real], t) })
    (forall(t: Real) { interval_contains(Real.0, Real.1, t) implies Real.0 <= uniform_mass_weighted(identity_fn[Real], t) }) = true
    eq_true_intro(forall(t: Real) { interval_contains(Real.0, Real.1, t) implies uniform_mass_weighted(identity_fn[Real], t) <= Real.1 })
    (forall(t: Real) { interval_contains(Real.0, Real.1, t) implies uniform_mass_weighted(identity_fn[Real], t) <= Real.1 }) = true
    Real.0 <= Real.1 and is_integrable(identity_fn[Real], Real.0, Real.1) and
        (forall(t: Real) { interval_contains(Real.0, Real.1, t) implies uniform_mass_weighted(identity_fn[Real], t) = identity_fn[Real](t) }) and
        (forall(t: Real) { interval_contains(Real.0, Real.1, t) implies Real.0 <= uniform_mass_weighted(identity_fn[Real], t) }) and
        (forall(t: Real) { interval_contains(Real.0, Real.1, t) implies uniform_mass_weighted(identity_fn[Real], t) <= Real.1 })
    integral_and_integrable_eq_on(uniform_mass_weighted(identity_fn[Real]), identity_fn[Real], Real.0, Real.1, Real.0, Real.1)
    is_integrable(uniform_mass_weighted(identity_fn[Real]), Real.0, Real.1) and
        integral(uniform_mass_weighted(identity_fn[Real]), Real.0, Real.1) = integral(identity_fn[Real], Real.0, Real.1)
    integral(uniform_mass_weighted(identity_fn[Real]), Real.0, Real.1) = integral(identity_fn[Real], Real.0, Real.1)
    uniform_expectation(identity_fn[Real]) = integral(identity_fn[Real], Real.0, Real.1)
    integral_identity(Real.0, Real.1)
    is_integrable(identity_fn[Real], Real.0, Real.1) and
        integral(identity_fn[Real], Real.0, Real.1) = (Real.1 * Real.1 - Real.0 * Real.0) * Real.one_half
    integral(identity_fn[Real], Real.0, Real.1) = (Real.1 * Real.1 - Real.0 * Real.0) * Real.one_half
    Real.1 * Real.1 = Real.1
    Real.0 * Real.0 = Real.0
    Real.1 * Real.1 - Real.0 * Real.0 = Real.1
    (Real.1 * Real.1 - Real.0 * Real.0) * Real.one_half = Real.one_half
    integral(identity_fn[Real], Real.0, Real.1) = Real.one_half
    uniform_expectation(identity_fn[Real]) = Real.one_half
}

/// The variance integrand of the identity under the uniform law on [0, 1].
define uniform_variance_integrand(x: Real) -> Real {
    centered_square_unit(x) * uniform_pdf(x)
}

/// The variance of a random variable under the uniform law on [0, 1].
define uniform_variance(rv: Real -> Real) -> Real {
    integral(uniform_variance_integrand, Real.0, Real.1)
}

/// The variance integrand agrees with the centered square on [0, 1].
lemma uniform_variance_integrand_eq_on_unit(t: Real) {
    interval_contains(Real.0, Real.1, t) implies uniform_variance_integrand(t) = centered_square_unit(t)
} by {
    if interval_contains(Real.0, Real.1, t) {
        uniform_pdf_one_on_unit(t)
        uniform_pdf(t) = Real.1
        uniform_variance_integrand(t) = centered_square_unit(t) * uniform_pdf(t)
        uniform_variance_integrand(t) = centered_square_unit(t) * Real.1
        centered_square_unit(t) * Real.1 = centered_square_unit(t)
        uniform_variance_integrand(t) = centered_square_unit(t)
    }
}

/// The square of a difference expands.
lemma sq_sub_expand(t: Real, a: Real) {
    (t - a) * (t - a) = t * t - (a * t + a * t) + a * a
} by {
    (t - a) * (t - a) = (t - a) * t - (t - a) * a
    (t - a) * t = t * t - a * t
    (t - a) * a = t * a - a * a
    (t - a) * (t - a) = t * t - a * t - (t * a - a * a)
    neg_distrib(t * a, -(a * a))
    -(t * a + -(a * a)) = -(t * a) + -(-(a * a))
    -(-(a * a)) = a * a
    -(t * a + -(a * a)) = -(t * a) + a * a
    t * a - a * a = t * a + -(a * a)
    -(t * a - a * a) = -(t * a) + a * a
    t * t - a * t - (t * a - a * a) = t * t - a * t + -(t * a - a * a)
    t * t - a * t + -(t * a - a * a) = t * t - a * t + (-(t * a) + a * a)
    t * t - a * t - (t * a - a * a) = t * t - a * t + (-(t * a) + a * a)
    t * t - a * t + (-(t * a) + a * a) = t * t - a * t - t * a + a * a
    t * t - a * t - t * a + a * a = t * t - (a * t + t * a) + a * a
    real_mul_comm(t, a)
    t * a = a * t
    a * t + t * a = a * t + a * t
    t * t - (a * t + t * a) + a * a = t * t - (a * t + a * t) + a * a
    (t - a) * (t - a) = t * t - (a * t + a * t) + a * a
}

/// The centered square is bounded above by one quarter on [0, 1].
lemma centered_square_upper_bound_unit(t: Real) {
    interval_contains(Real.0, Real.1, t) implies centered_square_unit(t) <= Real.one_half * Real.one_half
} by {
    if interval_contains(Real.0, Real.1, t) {
        centered_square_unit(t) = (t - Real.one_half) * (t - Real.one_half)
        interval_contains_right(Real.0, Real.1, t)
        t <= Real.1
        interval_contains_left(Real.0, Real.1, t)
        Real.0 <= t
        mul_le_mul_of_nonneg_right(t, Real.1, t)
        t * t <= t * Real.1
        t * Real.1 = t
        t * t <= t
        sq_sub_expand(t, Real.one_half)
        (t - Real.one_half) * (t - Real.one_half) = t * t - (Real.one_half * t + Real.one_half * t) + Real.one_half * Real.one_half
        Real.one_half * t + Real.one_half * t = (Real.one_half + Real.one_half) * t
        one_half_plus_one_half
        Real.one_half + Real.one_half = Real.1
        (Real.one_half + Real.one_half) * t = Real.1 * t
        Real.1 * t = t
        Real.one_half * t + Real.one_half * t = t
        (t - Real.one_half) * (t - Real.one_half) = t * t - t + Real.one_half * Real.one_half
        add_le_add_right(t * t, t, -t)
        t * t + -t <= t + -t
        t * t - t = t * t + -t
        t + -t = Real.0
        t * t - t <= Real.0
        add_le_add_right(t * t - t, Real.0, Real.one_half * Real.one_half)
        t * t - t + Real.one_half * Real.one_half <= Real.0 + Real.one_half * Real.one_half
        Real.0 + Real.one_half * Real.one_half = Real.one_half * Real.one_half
        (t - Real.one_half) * (t - Real.one_half) <= Real.one_half * Real.one_half
        centered_square_unit(t) <= Real.one_half * Real.one_half
    }
}

/// The variance integrand is bounded on [0, 1].
lemma uniform_variance_integrand_bounds_unit(t: Real) {
    interval_contains(Real.0, Real.1, t) implies (Real.0 <= uniform_variance_integrand(t) and uniform_variance_integrand(t) <= Real.one_half * Real.one_half)
} by {
    if interval_contains(Real.0, Real.1, t) {
        centered_square_unit(t) = (t - Real.one_half) * (t - Real.one_half)
        square_nonneg(t - Real.one_half)
        Real.0 <= (t - Real.one_half) * (t - Real.one_half)
        Real.0 <= centered_square_unit(t)
        uniform_variance_integrand_eq_on_unit(t)
        uniform_variance_integrand(t) = centered_square_unit(t)
        Real.0 <= uniform_variance_integrand(t)
        centered_square_upper_bound_unit(t)
        centered_square_unit(t) <= Real.one_half * Real.one_half
        uniform_variance_integrand(t) <= Real.one_half * Real.one_half
        Real.0 <= uniform_variance_integrand(t) and uniform_variance_integrand(t) <= Real.one_half * Real.one_half
    }
}

/// The variance of X under the uniform law on [0, 1] is one twelfth.
theorem uniform_variance_unit {
    uniform_variance(identity_fn[Real]) = (Real.1 / (Real.1 + Real.1 + Real.1)) * (Real.one_half * Real.one_half)
} by {
    zero_is_smaller_than_one[Real]
    Real.0 < Real.1
    lt_imp_lte(Real.0, Real.1)
    Real.0 <= Real.1
    integral_centered_square_unit_third
    is_integrable(centered_square_unit, Real.0, Real.1) and
        integral(centered_square_unit, Real.0, Real.1) = (Real.1 / (Real.1 + Real.1 + Real.1)) * (Real.one_half * Real.one_half)
    is_integrable(centered_square_unit, Real.0, Real.1)
    forall(t: Real) {
        uniform_variance_integrand_eq_on_unit(t)
        interval_contains(Real.0, Real.1, t) implies uniform_variance_integrand(t) = centered_square_unit(t)
    }
    forall(t: Real) { interval_contains(Real.0, Real.1, t) implies uniform_variance_integrand(t) = centered_square_unit(t) }
    forall(t: Real) {
        uniform_variance_integrand_bounds_unit(t)
        interval_contains(Real.0, Real.1, t) implies (Real.0 <= uniform_variance_integrand(t) and uniform_variance_integrand(t) <= Real.one_half * Real.one_half)
        interval_contains(Real.0, Real.1, t) implies Real.0 <= uniform_variance_integrand(t)
    }
    forall(t: Real) { interval_contains(Real.0, Real.1, t) implies Real.0 <= uniform_variance_integrand(t) }
    forall(t: Real) {
        uniform_variance_integrand_bounds_unit(t)
        interval_contains(Real.0, Real.1, t) implies (Real.0 <= uniform_variance_integrand(t) and uniform_variance_integrand(t) <= Real.one_half * Real.one_half)
        interval_contains(Real.0, Real.1, t) implies uniform_variance_integrand(t) <= Real.one_half * Real.one_half
    }
    forall(t: Real) { interval_contains(Real.0, Real.1, t) implies uniform_variance_integrand(t) <= Real.one_half * Real.one_half }
    eq_true_intro(forall(t: Real) { interval_contains(Real.0, Real.1, t) implies uniform_variance_integrand(t) = centered_square_unit(t) })
    (forall(t: Real) { interval_contains(Real.0, Real.1, t) implies uniform_variance_integrand(t) = centered_square_unit(t) }) = true
    eq_true_intro(forall(t: Real) { interval_contains(Real.0, Real.1, t) implies Real.0 <= uniform_variance_integrand(t) })
    (forall(t: Real) { interval_contains(Real.0, Real.1, t) implies Real.0 <= uniform_variance_integrand(t) }) = true
    eq_true_intro(forall(t: Real) { interval_contains(Real.0, Real.1, t) implies uniform_variance_integrand(t) <= Real.one_half * Real.one_half })
    (forall(t: Real) { interval_contains(Real.0, Real.1, t) implies uniform_variance_integrand(t) <= Real.one_half * Real.one_half }) = true
    Real.0 <= Real.1 and is_integrable(centered_square_unit, Real.0, Real.1) and
        (forall(t: Real) { interval_contains(Real.0, Real.1, t) implies uniform_variance_integrand(t) = centered_square_unit(t) }) and
        (forall(t: Real) { interval_contains(Real.0, Real.1, t) implies Real.0 <= uniform_variance_integrand(t) }) and
        (forall(t: Real) { interval_contains(Real.0, Real.1, t) implies uniform_variance_integrand(t) <= Real.one_half * Real.one_half })
    integral_and_integrable_eq_on(uniform_variance_integrand, centered_square_unit, Real.0, Real.1, Real.0, Real.one_half * Real.one_half)
    is_integrable(uniform_variance_integrand, Real.0, Real.1) and
        integral(uniform_variance_integrand, Real.0, Real.1) = integral(centered_square_unit, Real.0, Real.1)
    integral(uniform_variance_integrand, Real.0, Real.1) = integral(centered_square_unit, Real.0, Real.1)
    uniform_variance(identity_fn[Real]) = integral(centered_square_unit, Real.0, Real.1)
    integral_centered_square_unit_third
    is_integrable(centered_square_unit, Real.0, Real.1) and
        integral(centered_square_unit, Real.0, Real.1) = (Real.1 / (Real.1 + Real.1 + Real.1)) * (Real.one_half * Real.one_half)
    integral(centered_square_unit, Real.0, Real.1) = (Real.1 / (Real.1 + Real.1 + Real.1)) * (Real.one_half * Real.one_half)
    uniform_variance(identity_fn[Real]) = (Real.1 / (Real.1 + Real.1 + Real.1)) * (Real.one_half * Real.one_half)
}

/// The cumulative distribution function of the uniform law on [0, 1]: the
/// mass of [0, x].
define uniform_cdf_unit(x: Real) -> Real {
    integral(uniform_pdf, Real.0, x)
}

/// The constant one function is integrable on [0, x].
lemma const_one_integrable_upto(x: Real) {
    Real.0 <= x implies is_integrable(constant[Real, Real](Real.1), Real.0, x)
} by {
    if Real.0 <= x {
        forall(t: Real) {
            if Real.0 < t and t < x {
                constant[Real, Real](Real.1, t) = Real.1
            }
        }
        forall(t: Real) { Real.0 < t and t < x implies constant[Real, Real](Real.1, t) = Real.1 }
        forall(t: Real) {
            if interval_contains(Real.0, x, t) {
                constant[Real, Real](Real.1, t) = Real.1
                Real.1 <= constant[Real, Real](Real.1, t)
            }
        }
        forall(t: Real) { interval_contains(Real.0, x, t) implies Real.1 <= constant[Real, Real](Real.1, t) }
        forall(t: Real) {
            if interval_contains(Real.0, x, t) {
                constant[Real, Real](Real.1, t) = Real.1
                constant[Real, Real](Real.1, t) <= Real.1
            }
        }
        forall(t: Real) { interval_contains(Real.0, x, t) implies constant[Real, Real](Real.1, t) <= Real.1 }
        eq_true_intro(forall(t: Real) { Real.0 < t and t < x implies constant[Real, Real](Real.1, t) = Real.1 })
        (forall(t: Real) { Real.0 < t and t < x implies constant[Real, Real](Real.1, t) = Real.1 }) = true
        eq_true_intro(forall(t: Real) { interval_contains(Real.0, x, t) implies Real.1 <= constant[Real, Real](Real.1, t) })
        (forall(t: Real) { interval_contains(Real.0, x, t) implies Real.1 <= constant[Real, Real](Real.1, t) }) = true
        eq_true_intro(forall(t: Real) { interval_contains(Real.0, x, t) implies constant[Real, Real](Real.1, t) <= Real.1 })
        (forall(t: Real) { interval_contains(Real.0, x, t) implies constant[Real, Real](Real.1, t) <= Real.1 }) = true
        Real.0 <= x and
            (forall(t: Real) { Real.0 < t and t < x implies constant[Real, Real](Real.1, t) = Real.1 }) and
            (forall(t: Real) { interval_contains(Real.0, x, t) implies Real.1 <= constant[Real, Real](Real.1, t) }) and
            (forall(t: Real) { interval_contains(Real.0, x, t) implies constant[Real, Real](Real.1, t) <= Real.1 })
        interior_constant_integrable(constant[Real, Real](Real.1), Real.0, x, Real.1, Real.1, Real.1)
        is_integrable(constant[Real, Real](Real.1), Real.0, x)
    }
}


/// The integral of the constant one function over [0, x] is x.
lemma const_one_integral_upto(x: Real) {
    Real.0 <= x implies integral(constant[Real, Real](Real.1), Real.0, x) = x
} by {
    if Real.0 <= x {
        const_one_integrable_upto(x)
        is_integrable(constant[Real, Real](Real.1), Real.0, x)
        identity_function_is_continuous
        continuous(identity_fn[Real])
        derivative_fn_identity
        is_derivative_fn(identity_fn[Real], constant[Real, Real](Real.1))
        forall(t: Real) {
            if interval_contains(Real.0, x, t) {
                constant[Real, Real](Real.1, t) = Real.1
                Real.1 <= constant[Real, Real](Real.1, t)
            }
        }
        forall(t: Real) { interval_contains(Real.0, x, t) implies Real.1 <= constant[Real, Real](Real.1, t) }
        forall(t: Real) {
            if interval_contains(Real.0, x, t) {
                constant[Real, Real](Real.1, t) = Real.1
                constant[Real, Real](Real.1, t) <= Real.1
            }
        }
        forall(t: Real) { interval_contains(Real.0, x, t) implies constant[Real, Real](Real.1, t) <= Real.1 }
        eq_true_intro(continuous(identity_fn[Real]))
        (continuous(identity_fn[Real])) = true
        eq_true_intro(is_derivative_fn(identity_fn[Real], constant[Real, Real](Real.1)))
        (is_derivative_fn(identity_fn[Real], constant[Real, Real](Real.1))) = true
        eq_true_intro(forall(t: Real) { interval_contains(Real.0, x, t) implies Real.1 <= constant[Real, Real](Real.1, t) })
        (forall(t: Real) { interval_contains(Real.0, x, t) implies Real.1 <= constant[Real, Real](Real.1, t) }) = true
        eq_true_intro(forall(t: Real) { interval_contains(Real.0, x, t) implies constant[Real, Real](Real.1, t) <= Real.1 })
        (forall(t: Real) { interval_contains(Real.0, x, t) implies constant[Real, Real](Real.1, t) <= Real.1 }) = true
        Real.0 <= x and continuous(identity_fn[Real]) and
            is_derivative_fn(identity_fn[Real], constant[Real, Real](Real.1)) and
            is_integrable(constant[Real, Real](Real.1), Real.0, x) and
            (forall(t: Real) { interval_contains(Real.0, x, t) implies Real.1 <= constant[Real, Real](Real.1, t) }) and
            (forall(t: Real) { interval_contains(Real.0, x, t) implies constant[Real, Real](Real.1, t) <= Real.1 })
        ftc2_general(constant[Real, Real](Real.1), identity_fn[Real], Real.0, x, Real.1, Real.1)
        integral(constant[Real, Real](Real.1), Real.0, x) = identity_fn[Real](x) - identity_fn[Real](Real.0)
        identity_fn[Real](x) = x
        identity_fn[Real](Real.0) = Real.0
        identity_fn[Real](x) - identity_fn[Real](Real.0) = x - Real.0
        x - Real.0 = x
        integral(constant[Real, Real](Real.1), Real.0, x) = x
    }
}

/// The CDF of the uniform law on [0, 1] at x in [0, 1] is x.
theorem uniform_cdf_unit_value(x: Real) {
    interval_contains(Real.0, Real.1, x) implies uniform_cdf_unit(x) = x
} by {
    if interval_contains(Real.0, Real.1, x) {
        interval_contains_left(Real.0, Real.1, x)
        Real.0 <= x
        const_one_integrable_upto(x)
        is_integrable(constant[Real, Real](Real.1), Real.0, x)
        forall(t: Real) {
            if interval_contains(Real.0, x, t) {
                interval_contains_left(Real.0, x, t)
                Real.0 <= t
                interval_contains_right(Real.0, x, t)
                t <= x
                interval_contains_right(Real.0, Real.1, x)
                x <= Real.1
                lte_trans(t, x, Real.1)
                t <= Real.1
                interval_contains(Real.0, Real.1, t)
                uniform_pdf_eq_const_one_on_unit(t)
                uniform_pdf(t) = constant[Real, Real](Real.1, t)
            }
        }
        forall(t: Real) { interval_contains(Real.0, x, t) implies uniform_pdf(t) = constant[Real, Real](Real.1, t) }
        forall(t: Real) {
            if interval_contains(Real.0, x, t) {
                uniform_pdf_nonneg(t)
                Real.0 <= uniform_pdf(t)
            }
        }
        forall(t: Real) { interval_contains(Real.0, x, t) implies Real.0 <= uniform_pdf(t) }
        forall(t: Real) {
            if interval_contains(Real.0, x, t) {
                uniform_pdf_le_one(t)
                uniform_pdf(t) <= Real.1
            }
        }
        forall(t: Real) { interval_contains(Real.0, x, t) implies uniform_pdf(t) <= Real.1 }
        eq_true_intro(forall(t: Real) { interval_contains(Real.0, x, t) implies uniform_pdf(t) = constant[Real, Real](Real.1, t) })
        (forall(t: Real) { interval_contains(Real.0, x, t) implies uniform_pdf(t) = constant[Real, Real](Real.1, t) }) = true
        eq_true_intro(forall(t: Real) { interval_contains(Real.0, x, t) implies Real.0 <= uniform_pdf(t) })
        (forall(t: Real) { interval_contains(Real.0, x, t) implies Real.0 <= uniform_pdf(t) }) = true
        eq_true_intro(forall(t: Real) { interval_contains(Real.0, x, t) implies uniform_pdf(t) <= Real.1 })
        (forall(t: Real) { interval_contains(Real.0, x, t) implies uniform_pdf(t) <= Real.1 }) = true
        Real.0 <= x and is_integrable(constant[Real, Real](Real.1), Real.0, x) and
            (forall(t: Real) { interval_contains(Real.0, x, t) implies uniform_pdf(t) = constant[Real, Real](Real.1, t) }) and
            (forall(t: Real) { interval_contains(Real.0, x, t) implies Real.0 <= uniform_pdf(t) }) and
            (forall(t: Real) { interval_contains(Real.0, x, t) implies uniform_pdf(t) <= Real.1 })
        integral_and_integrable_eq_on(uniform_pdf, constant[Real, Real](Real.1), Real.0, x, Real.0, Real.1)
        is_integrable(uniform_pdf, Real.0, x) and
            integral(uniform_pdf, Real.0, x) = integral(constant[Real, Real](Real.1), Real.0, x)
        integral(uniform_pdf, Real.0, x) = integral(constant[Real, Real](Real.1), Real.0, x)
        uniform_cdf_unit(x) = integral(constant[Real, Real](Real.1), Real.0, x)
        const_one_integral_upto(x)
        integral(constant[Real, Real](Real.1), Real.0, x) = x
        uniform_cdf_unit(x) = x
    }
}

// ---------------------------------------------------------------------------
// The uniform law on a general interval [a, b] (future work)
// ---------------------------------------------------------------------------
//
// The same development goes through for the uniform law on [a, b] with
// a < b: the density is the constant 1 / (b - a) on [a, b] and zero outside.
// Its integral over [a, b] is one, the expectation is (a + b) / 2, the
// variance is (b - a)^2 / 12, and the cumulative distribution function on
// [a, b] is (x - a) / (b - a).  Each of these reduces, exactly as on [0, 1],
// to the integral of a constant function, computed by the fundamental
// theorem with the antiderivative x -> c * x (whose derivative follows from
// the product rule).
//
// The proofs are left for a follow-up because the public real interface is
// at its declaration limit (the facade admits no further exports), and the
// continuity rule for pointwise products needed by the c * x antiderivative
// is not exposed to this package.
