/// Tail, shift, and subsequence transport for Nat-eventual real sequence predicates.
///
/// This module intentionally does not define asymptotic notation. It packages
/// shallow transport lemmas needed by downstream sequence and asymptotics APIs.

from data.basic.functions import compose
from nat import Nat
from data.basic.relation_transport import predicate_pullback
from data.nat.nat_eventually import eventually_after_nat, eventually_after_nat_at,
    eventually_predicate_nat, eventually_predicate_nat_has_witness,
    eventually_predicate_nat_intro
from data.basic.set import Set
from real.real_field import Real
from real.sequence_set_membership import seq_eventually_in_real_set
from real.sequence_eventual_bridge import seq_eventually_in_real_set_iff_pullback_eventually,
    seq_eventually_equal_real, real_sequence_eq_predicate
from real.limits import tends_to_infinity, is_subsequence_index, subsequence,
    subsequence_eq_compose, subsequence_index_tends_to_infinity,
    add_subsequence_index

/// Pulling a Nat-eventual predicate back along an index map tending to infinity
/// preserves eventuality.
theorem eventually_predicate_nat_pullback_tends_to_infinity(p: Nat -> Bool, f: Nat -> Nat) {
    eventually_predicate_nat(p) and tends_to_infinity(f)
    implies eventually_predicate_nat(predicate_pullback(f, p))
} by {
    if eventually_predicate_nat(p) and tends_to_infinity(f) {
        eventually_predicate_nat_has_witness(p)
        let n0: Nat satisfy {
            eventually_after_nat(p, n0)
        }
        tends_to_infinity(f) = forall(big_n: Nat) {
            exists(m: Nat) {
                forall(n: Nat) {
                    m <= n implies big_n <= f(n)
                }
            }
        }
        let m0: Nat satisfy {
            forall(n: Nat) {
                m0 <= n implies n0 <= f(n)
            }
        }
        forall(n: Nat) {
            if m0 <= n {
                n0 <= f(n)
                eventually_after_nat_at(p, n0, f(n))
                p(f(n))
                predicate_pullback(f, p, n)
            }
        }
        eventually_after_nat(predicate_pullback(f, p), m0)
        eventually_predicate_nat_intro(predicate_pullback(f, p), m0)
        eventually_predicate_nat(predicate_pullback(f, p))
    }
}

/// Eventual real-set membership is preserved by reindexing along any map tending to infinity.
theorem seq_eventually_in_real_set_compose_tends_to_infinity(
    s: Set[Real],
    a: Nat -> Real,
    f: Nat -> Nat
) {
    seq_eventually_in_real_set(s, a) and tends_to_infinity(f)
    implies seq_eventually_in_real_set(s, compose(a, f))
} by {
    if seq_eventually_in_real_set(s, a) and tends_to_infinity(f) {
        seq_eventually_in_real_set_iff_pullback_eventually(s, a)
        eventually_predicate_nat(predicate_pullback(a, s.contains))
        eventually_predicate_nat_pullback_tends_to_infinity(predicate_pullback(a, s.contains), f)
        eventually_predicate_nat(predicate_pullback(f, predicate_pullback(a, s.contains)))
        forall(n: Nat) {
            predicate_pullback(f, predicate_pullback(a, s.contains), n) =
                predicate_pullback(a, s.contains, f(n))
            predicate_pullback(a, s.contains, f(n)) = s.contains(a(f(n)))
            compose(a, f)(n) = a(f(n))
            predicate_pullback(compose(a, f), s.contains, n) = s.contains(compose(a, f)(n))
            predicate_pullback(f, predicate_pullback(a, s.contains), n) =
                predicate_pullback(compose(a, f), s.contains, n)
        }
        predicate_pullback(f, predicate_pullback(a, s.contains)) =
            predicate_pullback(compose(a, f), s.contains)
        eventually_predicate_nat(predicate_pullback(compose(a, f), s.contains))
        seq_eventually_in_real_set_iff_pullback_eventually(s, compose(a, f))
        seq_eventually_in_real_set(s, compose(a, f))
    }
}

/// Eventual real-set membership is invariant under discarding a finite prefix of the sequence.
theorem seq_eventually_in_real_set_shift_add(s: Set[Real], a: Nat -> Real, k: Nat) {
    seq_eventually_in_real_set(s, a) implies seq_eventually_in_real_set(s, compose(a, k.add))
} by {
    if seq_eventually_in_real_set(s, a) {
        add_subsequence_index(k)
        subsequence_index_tends_to_infinity(k.add)
        tends_to_infinity(k.add)
        seq_eventually_in_real_set_compose_tends_to_infinity(s, a, k.add)
        seq_eventually_in_real_set(s, compose(a, k.add))
    }
}

/// Eventual real-set membership is preserved by a library subsequence index.
theorem seq_eventually_in_real_set_subsequence(
    s: Set[Real],
    a: Nat -> Real,
    f: Nat -> Nat
) {
    seq_eventually_in_real_set(s, a) and is_subsequence_index(f)
    implies seq_eventually_in_real_set(s, subsequence(a, f))
} by {
    if seq_eventually_in_real_set(s, a) and is_subsequence_index(f) {
        subsequence_index_tends_to_infinity(f)
        tends_to_infinity(f)
        seq_eventually_in_real_set_compose_tends_to_infinity(s, a, f)
        seq_eventually_in_real_set(s, compose(a, f))
        subsequence_eq_compose(a, f)
        seq_eventually_in_real_set(s, subsequence(a, f))
    }
}

/// Eventual real-set membership is preserved by a tail subsequence `n ↦ k + n`.
theorem seq_eventually_in_real_set_tail_subsequence(s: Set[Real], a: Nat -> Real, k: Nat) {
    seq_eventually_in_real_set(s, a) implies seq_eventually_in_real_set(s, subsequence(a, k.add))
} by {
    if seq_eventually_in_real_set(s, a) {
        add_subsequence_index(k)
        seq_eventually_in_real_set_subsequence(s, a, k.add)
        seq_eventually_in_real_set(s, subsequence(a, k.add))
    }
}

/// Eventual equality of real sequences is preserved by reindexing along any map tending to infinity.
theorem seq_eventually_equal_real_compose_tends_to_infinity(
    a: Nat -> Real,
    b: Nat -> Real,
    f: Nat -> Nat
) {
    seq_eventually_equal_real(a, b) and tends_to_infinity(f)
    implies seq_eventually_equal_real(compose(a, f), compose(b, f))
} by {
    if seq_eventually_equal_real(a, b) and tends_to_infinity(f) {
        eventually_predicate_nat_has_witness(real_sequence_eq_predicate(a, b))
        let n0: Nat satisfy {
            eventually_after_nat(real_sequence_eq_predicate(a, b), n0)
        }
        tends_to_infinity(f) = forall(big_n: Nat) {
            exists(m: Nat) {
                forall(n: Nat) {
                    m <= n implies big_n <= f(n)
                }
            }
        }
        let m0: Nat satisfy {
            forall(n: Nat) {
                m0 <= n implies n0 <= f(n)
            }
        }
        forall(n: Nat) {
            if m0 <= n {
                n0 <= f(n)
                eventually_after_nat_at(real_sequence_eq_predicate(a, b), n0, f(n))
                real_sequence_eq_predicate(a, b, f(n))
                a(f(n)) = b(f(n))
                compose(a, f)(n) = a(f(n))
                compose(b, f)(n) = b(f(n))
                compose(a, f)(n) = compose(b, f)(n)
                real_sequence_eq_predicate(compose(a, f), compose(b, f), n)
            }
        }
        eventually_after_nat(real_sequence_eq_predicate(compose(a, f), compose(b, f)), m0)
        eventually_predicate_nat_intro(real_sequence_eq_predicate(compose(a, f), compose(b, f)), m0)
        eventually_predicate_nat(real_sequence_eq_predicate(compose(a, f), compose(b, f)))
        seq_eventually_equal_real(compose(a, f), compose(b, f))
    }
}

/// Eventual equality of real sequences is invariant under discarding a finite prefix.
theorem seq_eventually_equal_real_shift_add(a: Nat -> Real, b: Nat -> Real, k: Nat) {
    seq_eventually_equal_real(a, b) implies
    seq_eventually_equal_real(compose(a, k.add), compose(b, k.add))
} by {
    if seq_eventually_equal_real(a, b) {
        add_subsequence_index(k)
        subsequence_index_tends_to_infinity(k.add)
        tends_to_infinity(k.add)
        seq_eventually_equal_real_compose_tends_to_infinity(a, b, k.add)
        seq_eventually_equal_real(compose(a, k.add), compose(b, k.add))
    }
}

/// Eventual equality of real sequences is preserved by a library subsequence index.
theorem seq_eventually_equal_real_subsequence(
    a: Nat -> Real,
    b: Nat -> Real,
    f: Nat -> Nat
) {
    seq_eventually_equal_real(a, b) and is_subsequence_index(f)
    implies seq_eventually_equal_real(subsequence(a, f), subsequence(b, f))
} by {
    if seq_eventually_equal_real(a, b) and is_subsequence_index(f) {
        subsequence_index_tends_to_infinity(f)
        tends_to_infinity(f)
        seq_eventually_equal_real_compose_tends_to_infinity(a, b, f)
        seq_eventually_equal_real(compose(a, f), compose(b, f))
        subsequence_eq_compose(a, f)
        subsequence_eq_compose(b, f)
        seq_eventually_equal_real(subsequence(a, f), subsequence(b, f))
    }
}

/// Eventual equality of real sequences is preserved by a tail subsequence `n ↦ k + n`.
theorem seq_eventually_equal_real_tail_subsequence(a: Nat -> Real, b: Nat -> Real, k: Nat) {
    seq_eventually_equal_real(a, b) implies
    seq_eventually_equal_real(subsequence(a, k.add), subsequence(b, k.add))
} by {
    if seq_eventually_equal_real(a, b) {
        add_subsequence_index(k)
        seq_eventually_equal_real_subsequence(a, b, k.add)
        seq_eventually_equal_real(subsequence(a, k.add), subsequence(b, k.add))
    }
}
