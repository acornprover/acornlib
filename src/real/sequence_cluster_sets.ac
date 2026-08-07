from nat import Nat
from data.basic.set import Set, subset_contains
from real.real_field import Real
from real.real_seq import converges_to
from real.sequence_set_membership import seq_eventually_in_real_set, seq_frequently_in_real_set,
    seq_in_real_set
from real.sequence_tail_sets import sequence_range_set, sequence_tail_meets_set, sequence_tail_set,
    seq_eventually_in_real_set_iff_tail_subset, seq_frequently_in_real_set_iff_tail_meets,
    seq_in_real_set_iff_range_subset
from real.topology import adherent_point_eps, closed_set_eq_closure, closure, closure_contains_eq,
    closure_mono, is_adherent_point_of_set, is_closed_set, is_interior_point, is_open_set,
    open_set_interior_point
from real.topology_sequence_tail_sets import convergent_sequence_limit_in_tail_closure,
    tail_closure_member_is_range_closure_member

/// True if a real number belongs to every tail closure of a sequence.
define sequence_cluster_contains(a: Nat -> Real, x: Real) -> Bool {
    forall(m: Nat) {
        closure(sequence_tail_set(a, m)).contains(x)
    }
}

/// The cluster set of a real sequence.
define sequence_cluster_set(a: Nat -> Real) -> Set[Real] {
    Set[Real].new(sequence_cluster_contains(a))
}

/// Membership in the cluster set is membership in every tail closure.
theorem sequence_cluster_set_contains_eq(a: Nat -> Real, x: Real) {
    sequence_cluster_set(a).contains(x) = forall(m: Nat) {
        closure(sequence_tail_set(a, m)).contains(x)
    }
}

/// A cluster point belongs to each tail closure.
theorem cluster_point_in_tail_closure(a: Nat -> Real, x: Real, m: Nat) {
    sequence_cluster_set(a).contains(x) implies closure(sequence_tail_set(a, m)).contains(x)
} by {
    if sequence_cluster_set(a).contains(x) {
        sequence_cluster_set_contains_eq(a, x)
        closure(sequence_tail_set(a, m)).contains(x)
    }
}

/// A point in every tail closure is a cluster point.
theorem sequence_cluster_set_contains_intro(a: Nat -> Real, x: Real) {
    forall(m: Nat) { closure(sequence_tail_set(a, m)).contains(x) }
    implies sequence_cluster_set(a).contains(x)
}

/// A cluster point belongs to the closure of the full sequence range.
theorem cluster_point_in_range_closure(a: Nat -> Real, x: Real) {
    sequence_cluster_set(a).contains(x) implies closure(sequence_range_set(a)).contains(x)
} by {
    if sequence_cluster_set(a).contains(x) {
        cluster_point_in_tail_closure(a, x, Nat.0)
        closure(sequence_tail_set(a, Nat.0)).contains(x)
        tail_closure_member_is_range_closure_member(a, Nat.0, x)
        closure(sequence_range_set(a)).contains(x)
    }
}

/// The cluster set is contained in the closure of the full sequence range.
theorem sequence_cluster_set_subset_range_closure(a: Nat -> Real) {
    sequence_cluster_set(a).subset(closure(sequence_range_set(a)))
} by {
    forall(x: Real) {
        if sequence_cluster_set(a).contains(x) {
            cluster_point_in_range_closure(a, x)
        }
    }
}

/// A convergent sequence has its limit in its cluster set.
theorem convergent_sequence_limit_in_cluster_set(a: Nat -> Real, x: Real) {
    converges_to(a, x) implies sequence_cluster_set(a).contains(x)
} by {
    if converges_to(a, x) {
        forall(m: Nat) {
            convergent_sequence_limit_in_tail_closure(a, m, x)
            closure(sequence_tail_set(a, m)).contains(x)
        }
        sequence_cluster_set(a).contains(x)
    }
}

/// The cluster set of a convergent sequence is inhabited by its limit.
theorem convergent_sequence_cluster_set_nonempty(a: Nat -> Real, x: Real) {
    converges_to(a, x) implies exists(y: Real) { sequence_cluster_set(a).contains(y) }
} by {
    if converges_to(a, x) {
        convergent_sequence_limit_in_cluster_set(a, x)
        exists(y: Real) {
            sequence_cluster_set(a).contains(y)
        }
    }
}

/// A cluster point in an open set forces every tail to meet that open set.
theorem cluster_point_open_set_tail_meets(a: Nat -> Real, x: Real, s: Set[Real]) {
    sequence_cluster_set(a).contains(x) and is_open_set(s) and s.contains(x)
    implies forall(m: Nat) { sequence_tail_meets_set(a, m, s) }
} by {
    if sequence_cluster_set(a).contains(x) and is_open_set(s) and s.contains(x) {
        open_set_interior_point(s, x)
        is_interior_point(s, x)
        let eps: Real satisfy {
            eps.is_positive and forall(y: Real) {
                y.is_close(x, eps) implies s.contains(y)
            }
        }
        forall(m: Nat) {
            cluster_point_in_tail_closure(a, x, m)
            closure(sequence_tail_set(a, m)).contains(x)
            closure_contains_eq(sequence_tail_set(a, m), x)
            is_adherent_point_of_set(sequence_tail_set(a, m), x)
            adherent_point_eps(sequence_tail_set(a, m), x, eps)
            let y: Real satisfy {
                sequence_tail_set(a, m).contains(y) and y.is_close(x, eps)
            }
            s.contains(y)
            exists(z: Real) {
                sequence_tail_set(a, m).contains(z) and s.contains(z)
            }
            sequence_tail_meets_set(a, m, s)
        }
    }
}

/// If a cluster point lies in an open set, the sequence is frequently in that set.
theorem cluster_point_open_set_seq_frequently_in(a: Nat -> Real, x: Real, s: Set[Real]) {
    sequence_cluster_set(a).contains(x) and is_open_set(s) and s.contains(x)
    implies seq_frequently_in_real_set(s, a)
} by {
    if sequence_cluster_set(a).contains(x) and is_open_set(s) and s.contains(x) {
        cluster_point_open_set_tail_meets(a, x, s)
        forall(m: Nat) { sequence_tail_meets_set(a, m, s) }
        seq_frequently_in_real_set_iff_tail_meets(s, a)
        seq_frequently_in_real_set(s, a)
    }
}

/// If some tail is contained in a closed set, every cluster point lies in that closed set.
theorem cluster_point_in_closed_set_of_tail_subset(
    a: Nat -> Real, x: Real, m: Nat, s: Set[Real]
) {
    sequence_cluster_set(a).contains(x) and sequence_tail_set(a, m).subset(s) and is_closed_set(s)
    implies s.contains(x)
} by {
    if sequence_cluster_set(a).contains(x) and sequence_tail_set(a, m).subset(s) and is_closed_set(s) {
        cluster_point_in_tail_closure(a, x, m)
        closure(sequence_tail_set(a, m)).contains(x)
        closure_mono(sequence_tail_set(a, m), s)
        closure(sequence_tail_set(a, m)).subset(closure(s))
        subset_contains(closure(sequence_tail_set(a, m)), closure(s), x)
        closure(s).contains(x)
        closed_set_eq_closure(s)
        s = closure(s)
        s.contains(x)
    }
}

/// If a sequence is eventually in a closed set, every cluster point lies in that set.
theorem cluster_set_subset_closed_set_of_eventually_in(
    a: Nat -> Real, s: Set[Real]
) {
    seq_eventually_in_real_set(s, a) and is_closed_set(s) implies sequence_cluster_set(a).subset(s)
} by {
    if seq_eventually_in_real_set(s, a) and is_closed_set(s) {
        seq_eventually_in_real_set_iff_tail_subset(s, a)
        let m: Nat satisfy {
            sequence_tail_set(a, m).subset(s)
        }
        forall(x: Real) {
            if sequence_cluster_set(a).contains(x) {
                cluster_point_in_closed_set_of_tail_subset(a, x, m, s)
                s.contains(x)
            }
        }
    }
}

/// If a sequence is contained in a closed set, every cluster point lies in that set.
theorem cluster_set_subset_closed_set_of_seq_in(a: Nat -> Real, s: Set[Real]) {
    seq_in_real_set(s, a) and is_closed_set(s) implies sequence_cluster_set(a).subset(s)
} by {
    if seq_in_real_set(s, a) and is_closed_set(s) {
        seq_in_real_set_iff_range_subset(s, a)
        sequence_range_set(a).subset(s)
        from real.sequence_tail_sets import sequence_tail_set_subset_range_set
        sequence_tail_set_subset_range_set(a, Nat.0)
        sequence_tail_set(a, Nat.0).subset(sequence_range_set(a))
from data.basic.set import subset_trans
        subset_trans[Real](sequence_tail_set(a, Nat.0), sequence_range_set(a), s)
        sequence_tail_set(a, Nat.0).subset(s)
        forall(x: Real) {
            if sequence_cluster_set(a).contains(x) {
                cluster_point_in_closed_set_of_tail_subset(a, x, Nat.0, s)
                s.contains(x)
            }
        }
    }
}
