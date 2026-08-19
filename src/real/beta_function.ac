/// The Beta function on natural parameters.
///
/// For natural numbers n and m the Beta integrand is t ↦ t^n·(1-t)^m on the
/// unit interval, and the Beta value is its integral:
///
///     beta(n, m) = integral of t^n·(1-t)^m over [0, 1].
///
/// The main results are:
///   - beta_anti_derivative:  the antiderivative of the Beta integrand is the
///     finite alternating sum
///     A(t) = Σ_{j=0}^{m} (-1)^j · C(m,j) · t^(n+1+j)/(n+1+j),
///   - beta_integrable:       the Beta integrand is integrable on [0, 1],
///   - beta_value:            beta(n, m) = beta_sum(n, m), the alternating
///     binomial sum Σ_{j=0}^{m} (-1)^j · C(m,j)/(n+1+j),
///   - beta_closed_value:     beta(n, m) = n!·m!/(n+m+1)!.
///
/// The value is computed through the fundamental theorem of calculus
/// (ftc2_general from integral_exp.ac): the integrand is paired with the
/// antiderivative A above, and integrability follows from the Lipschitz
/// criterion fn_integrable_gen from integral_trig.ac.  The closed form is the
/// classical alternating-binomial identity
///
///     Σ_{j=0}^{m} (-1)^j · C(m,j)/(n+1+j) = n!·m!/(n+m+1)!,
///
/// proved by induction on m using Pascal's identity.

from nat import Nat, from_nat, alt_induction, from_nat_add, from_nat_one, from_nat_zero, pow_add, add_comm, add_assoc, add_sub, add_cancels_right, factorial_step, factorial_zero, from_nat_mul, lt_suc, lte_add_left
from order import lte_trans, lt_imp_lte
from data.basic.functions import function_extensionality, identity_fn, function_eq_transport_predicate_rev, compose
from data.basic.function_algebra import pointwise_mul, pointwise_add
from data.basic.logic import eq_true_intro
from real.real_field import Real
from real.continuity_pow import pow_real_fn, continuous_pow_real_fn, pow_real_fn_suc
from real.continuity_base import continuous
from real.continuity_const_mul import continuous_const_mul_left
from real.continuity_pointwise_mul import continuous_pointwise_mul
from real.continuity_pointwise import continuous_pointwise_add
from real.continuity_composition import continuous_compose, constant_function_is_continuous
from real.continuity_affine import affine_real, continuous_affine_real
from real.calculus_api import is_derivative_fn, derivative_fn_const_mul, derivative_fn_add, derivative_fn_constant
from real.exp import pow_suc, one_pow, zero_pow_pos, mul_one_over, mul_frac_assoc, from_nat_real_nonneg
from real.harmonic import from_nat_suc_pos_real
from real.integral import integral, is_integrable, interval_contains, interval_contains_left, interval_contains_right, neg_lte_flip
from real.integral_exp import ftc2_general
from real.integral_trig import fn_integrable_gen
from real.derivative_continuity import div_mul_cancel_denominator
from real.real_field import div_cancel_common
from real.real_ring import mul_assoc, real_mul_comm, mul_abs, mul_distrib_left, mul_sub_distrib_left
from real.real_series import pow_nonneg, abs_pow, triangle_ineq
from real.derivative_trig import abs_of_nonneg
from real.real_base import abs_gte_zero, neg_distrib, sub_cancels
from ordered_field import mul_le_mul_of_nonneg_right, zero_is_smaller_than_one
from algebra.add_ordered_group import add_le_add_right, add_le_add
from algebra.semigroup import mul_fn
from algebra.add_semigroup import add_fn
from algebra.ring.ring import alternating_sign, alternating_sign_zero, alternating_sign_suc, alternating_sign_eq_neg_one_pow, mul_neg_left
from comm_ring import binomial, binomial_term
from combinatorics import binom, pascal_suc_unbounded, choose_zero, choose_out_of_bounds
from list import partial, partial_add, partial_scalar_mul, partial_shift_suc, partial_split_last, partial_pointwise_eq, partial_one, partial_zero
from real.power_integral import monomial_lipschitz_unit, pow_real_fn_le_one_unit, pow_real_fn_nonneg_unit, derivative_pow_real_suc, pow_deriv_suc
from real.trig import real_pow_mul_distrib
from real.trig_identities import div_sub_same_denom, div_add_same_denom

numerals Real
numerals Nat

// ---------------------------------------------------------------------------
// Definitions
// ---------------------------------------------------------------------------

/// The complement x ↦ 1 - x of the unit interval.
define unit_minus(x: Real) -> Real {
    affine_real(-Real.1, Real.1, x)
}

/// The Beta integrand t ↦ t^n·(1-t)^m with natural parameters.
define beta_fn(n: Nat, m: Nat, t: Real) -> Real {
    pow_real_fn(n, t) * pow_real_fn(m, unit_minus(t))
}

/// The coefficient (-1)^j · C(m,j) / (n+1+j) of the Beta antiderivative.
define beta_anti_coeff(n: Nat, m: Nat, j: Nat) -> Real {
    alternating_sign[Real](j) * from_nat[Real](m.binom(j)) / from_nat[Real](n.suc + j)
}

/// The j-th term of the Beta antiderivative, as a function of t:
/// t ↦ (-1)^j · C(m,j) · t^(n+1+j)/(n+1+j).
define beta_anti_term_fn(n: Nat, m: Nat, j: Nat, t: Real) -> Real {
    beta_anti_coeff(n, m, j) * pow_real_fn(n.suc + j, t)
}

/// The j-th term of the Beta antiderivative sequence at a fixed point t.
define beta_anti_seq(n: Nat, m: Nat, t: Real, j: Nat) -> Real {
    beta_anti_term_fn(n, m, j, t)
}

/// The Beta antiderivative: the alternating partial sum of the terms above.
define beta_anti(n: Nat, m: Nat, t: Real) -> Real {
    partial[Real](beta_anti_seq(n, m, t), m.suc)
}

/// The j-th term of the derivative of the Beta antiderivative, as a function
/// of t: t ↦ (-1)^j · C(m,j) · t^(n+j).
define beta_deriv_term_fn(n: Nat, m: Nat, j: Nat, t: Real) -> Real {
    alternating_sign[Real](j) * from_nat[Real](m.binom(j)) * pow_real_fn(n + j, t)
}

/// The j-th derivative term sequence at a fixed point t.
define beta_deriv_seq(n: Nat, m: Nat, t: Real, j: Nat) -> Real {
    beta_deriv_term_fn(n, m, j, t)
}

/// The pointwise derivative of the Beta antiderivative.
define beta_anti_deriv(n: Nat, m: Nat, t: Real) -> Real {
    partial[Real](beta_deriv_seq(n, m, t), m.suc)
}

/// The j-th term of the alternating binomial sum
/// Σ_{j=0}^{m} (-1)^j · C(m,j)/(n+1+j).
define beta_sum_term(n: Nat, m: Nat, j: Nat) -> Real {
    alternating_sign[Real](j) * from_nat[Real](m.binom(j)) / from_nat[Real](n.suc + j)
}

/// The alternating binomial sum appearing as the value of the Beta function.
define beta_sum(n: Nat, m: Nat) -> Real {
    partial[Real](beta_sum_term(n, m), m.suc)
}

/// The closed form n!·m!/(n+m+1)! of the Beta function.
define beta_closed(n: Nat, m: Nat) -> Real {
    from_nat[Real](n.factorial) * from_nat[Real](m.factorial) / from_nat[Real]((n + m + Nat.1).factorial)
}

/// The Beta function with natural parameters: the integral of t^n·(1-t)^m
/// over [0, 1].
define beta(n: Nat, m: Nat) -> Real {
    integral(beta_fn(n, m), Real.0, Real.1)
}

// ---------------------------------------------------------------------------
// The t-dependent partial sum of a family of functions
// ---------------------------------------------------------------------------

/// The family f applied pointwise at t, as a sequence in the index.
define family_at(f: Nat -> Real -> Real, t: Real, j: Nat) -> Real {
    f(j)(t)
}

/// The t-dependent partial sum of the family f: Σ_{j<m} f(j, t).
define partial_of_family(f: Nat -> Real -> Real, m: Nat, t: Real) -> Real {
    partial[Real](family_at(f, t), m)
}

/// The pointwise derivative of a partial sum of differentiable terms is the
/// partial sum of the derivatives.
theorem family_partial_derivative(f: Nat -> Real -> Real, df: Nat -> Real -> Real, m: Nat) {
    (forall(j: Nat) { is_derivative_fn(f(j), df(j)) }) implies
    is_derivative_fn(partial_of_family(f, m), partial_of_family(df, m))
} by {
    if forall(j: Nat) { is_derivative_fn(f(j), df(j)) } {
        define p(x: Nat) -> Bool {
            is_derivative_fn(partial_of_family(f, x), partial_of_family(df, x))
        }

        // Base case: the empty partial sum is the constant zero function.
        forall(t: Real) {
            partial_of_family(f, Nat.0, t) = partial[Real](family_at(f, t), Nat.0)
            partial_zero(family_at(f, t))
            partial[Real](family_at(f, t), Nat.0) = Real.0
            constant[Real, Real](Real.0, t) = Real.0
            partial_of_family(f, Nat.0, t) = constant[Real, Real](Real.0, t)
        }
        function_extensionality(partial_of_family(f, Nat.0), constant[Real, Real](Real.0))
        partial_of_family(f, Nat.0) = constant[Real, Real](Real.0)
        forall(t: Real) {
            partial_of_family(df, Nat.0, t) = partial[Real](family_at(df, t), Nat.0)
            partial_zero(family_at(df, t))
            partial[Real](family_at(df, t), Nat.0) = Real.0
            constant[Real, Real](Real.0, t) = Real.0
            partial_of_family(df, Nat.0, t) = constant[Real, Real](Real.0, t)
        }
        function_extensionality(partial_of_family(df, Nat.0), constant[Real, Real](Real.0))
        partial_of_family(df, Nat.0) = constant[Real, Real](Real.0)
        derivative_fn_constant(Real.0)
        is_derivative_fn(constant[Real, Real](Real.0), constant[Real, Real](Real.0))
        function_eq_transport_predicate_rev[Real, Real](
            function(h: Real -> Real) { is_derivative_fn(h, constant[Real, Real](Real.0)) },
            partial_of_family(f, Nat.0), constant[Real, Real](Real.0))
        is_derivative_fn(partial_of_family(f, Nat.0), constant[Real, Real](Real.0))
        function_eq_transport_predicate_rev[Real, Real](
            function(h: Real -> Real) { is_derivative_fn(partial_of_family(f, Nat.0), h) },
            partial_of_family(df, Nat.0), constant[Real, Real](Real.0))
        is_derivative_fn(partial_of_family(f, Nat.0), partial_of_family(df, Nat.0))
        p(Nat.0)

        // Inductive step: partial[Real](f(t), x+1) = partial[Real](f(t), x) + f(x)(t).
        forall(x: Nat) {
            if p(x) {
                is_derivative_fn(partial_of_family(f, x), partial_of_family(df, x))
                is_derivative_fn(f(x), df(x))
                derivative_fn_add(partial_of_family(f, x), f(x), partial_of_family(df, x), df(x))
                is_derivative_fn(
                    pointwise_add(partial_of_family(f, x), f(x)),
                    pointwise_add(partial_of_family(df, x), df(x)))
                forall(t: Real) {
                    partial_of_family(f, x.suc, t) = partial[Real](family_at(f, t), x.suc)
                    partial_split_last(family_at(f, t), x)
                    partial[Real](family_at(f, t), x.suc) = partial[Real](family_at(f, t), x) + family_at(f, t)(x)
                    partial_of_family(f, x, t) = partial[Real](family_at(f, t), x)
                    pointwise_add(partial_of_family(f, x), f(x), t) =
                        partial_of_family(f, x, t) + f(x, t)
                    family_at(f, t)(x) = f(x, t)
                    partial_of_family(f, x.suc, t) = pointwise_add(partial_of_family(f, x), f(x), t)
                }
                function_extensionality(partial_of_family(f, x.suc),
                    pointwise_add(partial_of_family(f, x), f(x)))
                forall(t: Real) {
                    partial_of_family(df, x.suc, t) = partial[Real](family_at(df, t), x.suc)
                    partial_split_last(family_at(df, t), x)
                    partial[Real](family_at(df, t), x.suc) = partial[Real](family_at(df, t), x) + family_at(df, t)(x)
                    partial_of_family(df, x, t) = partial[Real](family_at(df, t), x)
                    pointwise_add(partial_of_family(df, x), df(x), t) =
                        partial_of_family(df, x, t) + df(x, t)
                    family_at(df, t)(x) = df(x, t)
                    partial_of_family(df, x.suc, t) = pointwise_add(partial_of_family(df, x), df(x), t)
                }
                function_extensionality(partial_of_family(df, x.suc),
                    pointwise_add(partial_of_family(df, x), df(x)))
                function_eq_transport_predicate_rev[Real, Real](
                    function(h: Real -> Real) {
                        is_derivative_fn(h, pointwise_add(partial_of_family(df, x), df(x)))
                    },
                    partial_of_family(f, x.suc),
                    pointwise_add(partial_of_family(f, x), f(x)))
                is_derivative_fn(partial_of_family(f, x.suc),
                    pointwise_add(partial_of_family(df, x), df(x)))
                function_eq_transport_predicate_rev[Real, Real](
                    function(h: Real -> Real) {
                        is_derivative_fn(partial_of_family(f, x.suc), h)
                    },
                    partial_of_family(df, x.suc),
                    pointwise_add(partial_of_family(df, x), df(x)))
                is_derivative_fn(partial_of_family(f, x.suc), partial_of_family(df, x.suc))
                p(x.suc)
            }
        }

        p(Nat.0) and forall(x: Nat) { p(x) implies p(x.suc) }
        alt_induction(p)
        forall(x: Nat) { p(x) }
        p(m)
        is_derivative_fn(partial_of_family(f, m), partial_of_family(df, m))
    }
}

/// The partial sum of a family of continuous terms is continuous.
theorem family_partial_continuous(f: Nat -> Real -> Real, m: Nat) {
    (forall(j: Nat) { continuous(f(j)) }) implies
    continuous(partial_of_family(f, m))
} by {
    if forall(j: Nat) { continuous(f(j)) } {
        define p(x: Nat) -> Bool {
            continuous(partial_of_family(f, x))
        }

        // Base case: the empty partial sum is the constant zero function.
        forall(t: Real) {
            partial_of_family(f, Nat.0, t) = partial[Real](family_at(f, t), Nat.0)
            partial_zero(family_at(f, t))
            partial[Real](family_at(f, t), Nat.0) = Real.0
            constant[Real, Real](Real.0, t) = Real.0
            partial_of_family(f, Nat.0, t) = constant[Real, Real](Real.0, t)
        }
        function_extensionality(partial_of_family(f, Nat.0), constant[Real, Real](Real.0))
        partial_of_family(f, Nat.0) = constant[Real, Real](Real.0)
        constant_function_is_continuous(Real.0)
        continuous(constant[Real, Real](Real.0))
        function_eq_transport_predicate_rev[Real, Real](
            function(h: Real -> Real) { continuous(h) },
            partial_of_family(f, Nat.0), constant[Real, Real](Real.0))
        continuous(partial_of_family(f, Nat.0))
        p(Nat.0)

        // Inductive step: partial[Real](f(t), x+1) = partial[Real](f(t), x) + f(x)(t).
        forall(x: Nat) {
            if p(x) {
                continuous(partial_of_family(f, x))
                continuous(f(x))
                continuous_pointwise_add(partial_of_family(f, x), f(x))
                continuous(pointwise_add(partial_of_family(f, x), f(x)))
                forall(t: Real) {
                    partial_of_family(f, x.suc, t) = partial[Real](family_at(f, t), x.suc)
                    partial_split_last(family_at(f, t), x)
                    partial[Real](family_at(f, t), x.suc) = partial[Real](family_at(f, t), x) + family_at(f, t)(x)
                    partial_of_family(f, x, t) = partial[Real](family_at(f, t), x)
                    pointwise_add(partial_of_family(f, x), f(x), t) =
                        partial_of_family(f, x, t) + f(x, t)
                    family_at(f, t)(x) = f(x, t)
                    partial_of_family(f, x.suc, t) = pointwise_add(partial_of_family(f, x), f(x), t)
                }
                function_extensionality(partial_of_family(f, x.suc),
                    pointwise_add(partial_of_family(f, x), f(x)))
                function_eq_transport_predicate_rev[Real, Real](
                    function(h: Real -> Real) { continuous(h) },
                    partial_of_family(f, x.suc),
                    pointwise_add(partial_of_family(f, x), f(x)))
                continuous(partial_of_family(f, x.suc))
                p(x.suc)
            }
        }

        p(Nat.0) and forall(x: Nat) { p(x) implies p(x.suc) }
        alt_induction(p)
        forall(x: Nat) { p(x) }
        p(m)
        continuous(partial_of_family(f, m))
    }
}

// ---------------------------------------------------------------------------
// The derivative of the Beta antiderivative
// ---------------------------------------------------------------------------

/// The coefficient times the successor of its denominator is the alternating
/// binomial coefficient.
lemma beta_anti_coeff_mul_denom(n: Nat, m: Nat, j: Nat) {
    beta_anti_coeff(n, m, j) * from_nat[Real](n.suc + j) =
        alternating_sign[Real](j) * from_nat[Real](m.binom(j))
} by {
    beta_anti_coeff(n, m, j) =
        alternating_sign[Real](j) * from_nat[Real](m.binom(j)) / from_nat[Real](n.suc + j)
    from_nat_suc_pos_real(n + j)
    from_nat[Real]((n + j).suc) > Real.0
    from_nat[Real]((n + j).suc) != Real.0
    (n + j).suc = n.suc + j
    from_nat[Real](n.suc + j) != Real.0
    div_mul_cancel_denominator(alternating_sign[Real](j) * from_nat[Real](m.binom(j)), from_nat[Real](n.suc + j))
    (alternating_sign[Real](j) * from_nat[Real](m.binom(j)) / from_nat[Real](n.suc + j)) *
        from_nat[Real](n.suc + j) = alternating_sign[Real](j) * from_nat[Real](m.binom(j))
    beta_anti_coeff(n, m, j) * from_nat[Real](n.suc + j) =
        alternating_sign[Real](j) * from_nat[Real](m.binom(j))
}

/// The coefficient times the derived power is the derivative term.
lemma beta_anti_term_const_mul_deriv(n: Nat, m: Nat, j: Nat) {
    forall(t: Real) {
        pointwise_mul[Real, Real](
            constant[Real, Real](beta_anti_coeff(n, m, j)),
            pow_deriv_suc(n + j), t) = beta_deriv_term_fn(n, m, j, t)
    }
} by {
    forall(t: Real) {
        pointwise_mul[Real, Real](
            constant[Real, Real](beta_anti_coeff(n, m, j)),
            pow_deriv_suc(n + j), t) =
            constant[Real, Real](beta_anti_coeff(n, m, j), t) * pow_deriv_suc(n + j, t)
        constant[Real, Real](beta_anti_coeff(n, m, j), t) = beta_anti_coeff(n, m, j)
        pow_deriv_suc(n + j, t) = from_nat[Real]((n + j).suc) * pow_real_fn(n + j, t)
        (n + j).suc = n.suc + j
        pow_deriv_suc(n + j, t) = from_nat[Real](n.suc + j) * pow_real_fn(n + j, t)
        beta_anti_coeff(n, m, j) * (from_nat[Real](n.suc + j) * pow_real_fn(n + j, t)) =
            (beta_anti_coeff(n, m, j) * from_nat[Real](n.suc + j)) * pow_real_fn(n + j, t)
        beta_anti_coeff_mul_denom(n, m, j)
        (beta_anti_coeff(n, m, j) * from_nat[Real](n.suc + j)) * pow_real_fn(n + j, t) =
            (alternating_sign[Real](j) * from_nat[Real](m.binom(j))) * pow_real_fn(n + j, t)
        beta_deriv_term_fn(n, m, j, t) =
            alternating_sign[Real](j) * from_nat[Real](m.binom(j)) * pow_real_fn(n + j, t)
        pointwise_mul[Real, Real](
            constant[Real, Real](beta_anti_coeff(n, m, j)),
            pow_deriv_suc(n + j), t) = beta_deriv_term_fn(n, m, j, t)
    }
    function_extensionality(
        pointwise_mul[Real, Real](
            constant[Real, Real](beta_anti_coeff(n, m, j)),
            pow_deriv_suc(n + j)),
        beta_deriv_term_fn(n, m, j))
}

/// The j-th antiderivative term is the coefficient times the power function.
lemma beta_anti_term_fn_const_mul_form(n: Nat, m: Nat, j: Nat) {
    beta_anti_term_fn(n, m, j) =
        pointwise_mul[Real, Real](
            constant[Real, Real](beta_anti_coeff(n, m, j)),
            pow_real_fn(n.suc + j))
} by {
    forall(t: Real) {
        beta_anti_term_fn(n, m, j, t) = beta_anti_coeff(n, m, j) * pow_real_fn(n.suc + j, t)
        pointwise_mul[Real, Real](
            constant[Real, Real](beta_anti_coeff(n, m, j)),
            pow_real_fn(n.suc + j), t) =
            constant[Real, Real](beta_anti_coeff(n, m, j), t) * pow_real_fn(n.suc + j, t)
        constant[Real, Real](beta_anti_coeff(n, m, j), t) = beta_anti_coeff(n, m, j)
        beta_anti_term_fn(n, m, j, t) =
            pointwise_mul[Real, Real](
                constant[Real, Real](beta_anti_coeff(n, m, j)),
                pow_real_fn(n.suc + j), t)
    }
    function_extensionality(beta_anti_term_fn(n, m, j),
        pointwise_mul[Real, Real](
            constant[Real, Real](beta_anti_coeff(n, m, j)),
            pow_real_fn(n.suc + j)))
}

/// The derivative of the j-th Beta antiderivative term is the j-th derivative
/// term.
theorem beta_anti_term_derivative(n: Nat, m: Nat, j: Nat) {
    is_derivative_fn(beta_anti_term_fn(n, m, j), beta_deriv_term_fn(n, m, j))
} by {
    derivative_pow_real_suc(n + j)
    is_derivative_fn(pow_real_fn((n + j).suc), pow_deriv_suc(n + j))
    derivative_fn_const_mul(beta_anti_coeff(n, m, j), pow_real_fn(n.suc + j), pow_deriv_suc(n + j))
    is_derivative_fn(
        pointwise_mul[Real, Real](
            constant[Real, Real](beta_anti_coeff(n, m, j)),
            pow_real_fn(n.suc + j)),
        pointwise_mul[Real, Real](
            constant[Real, Real](beta_anti_coeff(n, m, j)),
            pow_deriv_suc(n + j)))
    beta_anti_coeff_mul_denom(n, m, j)
    beta_anti_coeff(n, m, j) * from_nat[Real](n.suc + j) =
        alternating_sign[Real](j) * from_nat[Real](m.binom(j))
    beta_anti_term_const_mul_deriv(n, m, j)
    pointwise_mul[Real, Real](
        constant[Real, Real](beta_anti_coeff(n, m, j)),
        pow_deriv_suc(n + j)) = beta_deriv_term_fn(n, m, j)
    beta_anti_term_fn_const_mul_form(n, m, j)
    beta_anti_term_fn(n, m, j) =
        pointwise_mul[Real, Real](
            constant[Real, Real](beta_anti_coeff(n, m, j)),
            pow_real_fn(n.suc + j))
    function_eq_transport_predicate_rev[Real, Real](
        function(h: Real -> Real) { is_derivative_fn(h, beta_deriv_term_fn(n, m, j)) },
        beta_anti_term_fn(n, m, j),
        pointwise_mul[Real, Real](
            constant[Real, Real](beta_anti_coeff(n, m, j)),
            pow_real_fn(n.suc + j)))
    is_derivative_fn(beta_anti_term_fn(n, m, j), beta_deriv_term_fn(n, m, j))
}



/// The Beta antiderivative is the t-dependent partial sum of its terms.
lemma beta_anti_eq_family(n: Nat, m: Nat) {
    beta_anti(n, m) = partial_of_family(beta_anti_term_fn(n, m), m.suc)
} by {
    forall(t: Real) {
        beta_anti(n, m, t) = partial[Real](beta_anti_seq(n, m, t), m.suc)
        forall(k: Nat) {
            beta_anti_seq(n, m, t, k) = beta_anti_term_fn(n, m, k, t)
        }
        partial_pointwise_eq(beta_anti_seq(n, m, t), family_at(beta_anti_term_fn(n, m), t), m.suc)
        partial[Real](beta_anti_seq(n, m, t), m.suc) = partial[Real](family_at(beta_anti_term_fn(n, m), t), m.suc)
        partial_of_family(beta_anti_term_fn(n, m), m.suc, t) =
            partial[Real](family_at(beta_anti_term_fn(n, m), t), m.suc)
        beta_anti(n, m, t) = partial_of_family(beta_anti_term_fn(n, m), m.suc, t)
    }
    function_extensionality(beta_anti(n, m), partial_of_family(beta_anti_term_fn(n, m), m.suc))
}

/// The pointwise derivative of the Beta antiderivative is the t-dependent
/// partial sum of the derivative terms.
lemma beta_anti_deriv_eq_family(n: Nat, m: Nat) {
    beta_anti_deriv(n, m) = partial_of_family(beta_deriv_term_fn(n, m), m.suc)
} by {
    forall(t: Real) {
        beta_anti_deriv(n, m, t) = partial[Real](beta_deriv_seq(n, m, t), m.suc)
        forall(k: Nat) {
            beta_deriv_seq(n, m, t, k) = beta_deriv_term_fn(n, m, k, t)
        }
        partial_pointwise_eq(beta_deriv_seq(n, m, t), family_at(beta_deriv_term_fn(n, m), t), m.suc)
        partial[Real](beta_deriv_seq(n, m, t), m.suc) = partial[Real](family_at(beta_deriv_term_fn(n, m), t), m.suc)
        partial_of_family(beta_deriv_term_fn(n, m), m.suc, t) =
            partial[Real](family_at(beta_deriv_term_fn(n, m), t), m.suc)
        beta_anti_deriv(n, m, t) = partial_of_family(beta_deriv_term_fn(n, m), m.suc, t)
    }
    function_extensionality(beta_anti_deriv(n, m), partial_of_family(beta_deriv_term_fn(n, m), m.suc))
}

/// The derivative of the Beta antiderivative is the Beta antiderivative
/// derivative function.
theorem beta_anti_derivative(n: Nat, m: Nat) {
    is_derivative_fn(beta_anti(n, m), beta_anti_deriv(n, m))
} by {
    forall(j: Nat) {
        beta_anti_term_derivative(n, m, j)
        is_derivative_fn(beta_anti_term_fn(n, m, j), beta_deriv_term_fn(n, m, j))
    }
    family_partial_derivative(beta_anti_term_fn(n, m), beta_deriv_term_fn(n, m), m.suc)
    is_derivative_fn(
        partial_of_family(beta_anti_term_fn(n, m), m.suc),
        partial_of_family(beta_deriv_term_fn(n, m), m.suc))
    beta_anti_eq_family(n, m)
    beta_anti(n, m) = partial_of_family(beta_anti_term_fn(n, m), m.suc)
    beta_anti_deriv_eq_family(n, m)
    beta_anti_deriv(n, m) = partial_of_family(beta_deriv_term_fn(n, m), m.suc)
    function_eq_transport_predicate_rev[Real, Real](
        function(h: Real -> Real) { is_derivative_fn(h, partial_of_family(beta_deriv_term_fn(n, m), m.suc)) },
        beta_anti(n, m), partial_of_family(beta_anti_term_fn(n, m), m.suc))
    is_derivative_fn(beta_anti(n, m), partial_of_family(beta_deriv_term_fn(n, m), m.suc))
    function_eq_transport_predicate_rev[Real, Real](
        function(h: Real -> Real) { is_derivative_fn(beta_anti(n, m), h) },
        beta_anti_deriv(n, m), partial_of_family(beta_deriv_term_fn(n, m), m.suc))
    is_derivative_fn(beta_anti(n, m), beta_anti_deriv(n, m))
}

/// The Beta antiderivative is continuous.
theorem beta_anti_continuous(n: Nat, m: Nat) {
    continuous(beta_anti(n, m))
} by {
    forall(j: Nat) {
        continuous_pow_real_fn(n.suc + j)
        continuous(pow_real_fn(n.suc + j))
        continuous_const_mul_left(beta_anti_coeff(n, m, j), pow_real_fn(n.suc + j))
        continuous(pointwise_mul[Real, Real](
            constant[Real, Real](beta_anti_coeff(n, m, j)),
            pow_real_fn(n.suc + j)))
        beta_anti_term_fn_const_mul_form(n, m, j)
        function_eq_transport_predicate_rev[Real, Real](
            function(h: Real -> Real) { continuous(h) },
            beta_anti_term_fn(n, m, j),
            pointwise_mul[Real, Real](
                constant[Real, Real](beta_anti_coeff(n, m, j)),
                pow_real_fn(n.suc + j)))
        continuous(beta_anti_term_fn(n, m, j))
    }
    family_partial_continuous(beta_anti_term_fn(n, m), m.suc)
    continuous(partial_of_family(beta_anti_term_fn(n, m), m.suc))
    beta_anti_eq_family(n, m)
    function_eq_transport_predicate_rev[Real, Real](
        function(h: Real -> Real) { continuous(h) },
        beta_anti(n, m), partial_of_family(beta_anti_term_fn(n, m), m.suc))
    continuous(beta_anti(n, m))
}

// ---------------------------------------------------------------------------
// The Beta integrand is the derivative of the antiderivative
// ---------------------------------------------------------------------------

// ---------------------------------------------------------------------------
// The Beta integrand is the derivative of the antiderivative
// ---------------------------------------------------------------------------

/// The j-th term of the alternating binomial power sum at a point t.
define beta_pow_seq(m: Nat, t: Real, j: Nat) -> Real {
    alternating_sign[Real](j) * from_nat[Real](m.binom(j)) * pow_real_fn(j, t)
}

/// The binomial term of (1 - t)^m agrees pointwise with the alternating power
/// term.
lemma beta_binomial_term_eq_pow(m: Nat, t: Real, k: Nat) {
    binomial_term[Real](-t, Real.1, m, k) =
        alternating_sign[Real](k) * from_nat[Real](m.binom(k)) * pow_real_fn(k, t)
} by {
    binomial_term[Real](-t, Real.1, m, k) =
        from_nat[Real](m.binom(k)) * (-t).pow(k) * Real.1.pow(m - k)
    -t = (-Real.1) * t
    real_pow_mul_distrib(-Real.1, t, k)
    ((-Real.1) * t).pow(k) = (-Real.1).pow(k) * t.pow(k)
    (-t).pow(k) = (-Real.1).pow(k) * t.pow(k)
    alternating_sign_eq_neg_one_pow[Real](k)
    alternating_sign[Real](k) = (-Real.1).pow(k)
    (-t).pow(k) = alternating_sign[Real](k) * t.pow(k)
    one_pow[Real](m - k)
    Real.1.pow(m - k) = Real.1
    pow_real_fn(k, t) = t.pow(k)
    from_nat[Real](m.binom(k)) * (alternating_sign[Real](k) * t.pow(k)) * Real.1 =
        alternating_sign[Real](k) * from_nat[Real](m.binom(k)) * pow_real_fn(k, t)
    binomial_term[Real](-t, Real.1, m, k) =
        alternating_sign[Real](k) * from_nat[Real](m.binom(k)) * pow_real_fn(k, t)
}

/// The alternating binomial power sum at a point t is the power of (1 - t):
/// Σ_{j=0}^{m} (-1)^j · C(m,j) · t^j = (1-t)^m.
theorem beta_alternating_pow(m: Nat) {
    forall(t: Real) {
        partial[Real](beta_pow_seq(m, t), m.suc) = pow_real_fn(m, unit_minus(t))
    }
} by {
    forall(t: Real) {
        binomial[Real](-t, Real.1, m)
        (-t + Real.1).pow(m) = partial[Real](binomial_term[Real](-t, Real.1, m), m.suc)
        pow_real_fn(m, unit_minus(t)) = unit_minus(t).pow(m)
        unit_minus(t) = Real.1 - t
        Real.1 - t = -t + Real.1
        pow_real_fn(m, unit_minus(t)) = (-t + Real.1).pow(m)
        forall(k: Nat) {
            beta_binomial_term_eq_pow(m, t, k)
            binomial_term[Real](-t, Real.1, m, k) = beta_pow_seq(m, t, k)
        }
        partial_pointwise_eq(binomial_term[Real](-t, Real.1, m), beta_pow_seq(m, t), m.suc)
        partial[Real](binomial_term[Real](-t, Real.1, m), m.suc) = partial[Real](beta_pow_seq(m, t), m.suc)
        pow_real_fn(m, unit_minus(t)) = partial[Real](beta_pow_seq(m, t), m.suc)
    }
}

/// The pointwise derivative of the Beta antiderivative is the Beta integrand:
/// Σ_{j=0}^{m} (-1)^j · C(m,j) · t^(n+j) = t^n·(1-t)^m.
theorem beta_anti_deriv_eq_beta_fn(n: Nat, m: Nat) {
    beta_anti_deriv(n, m) = beta_fn(n, m)
} by {
    forall(t: Real) {
        beta_anti_deriv(n, m, t) = partial[Real](beta_deriv_seq(n, m, t), m.suc)
        forall(k: Nat) {
            beta_deriv_seq(n, m, t, k) = beta_deriv_term_fn(n, m, k, t)
        }
        partial_pointwise_eq(beta_deriv_seq(n, m, t),
            function(j: Nat) {
                alternating_sign[Real](j) * from_nat[Real](m.binom(j)) * pow_real_fn(n + j, t)
            }, m.suc)
        partial[Real](beta_deriv_seq(n, m, t), m.suc) =
            partial[Real](
                function(j: Nat) {
                    alternating_sign[Real](j) * from_nat[Real](m.binom(j)) * pow_real_fn(n + j, t)
                }, m.suc)
        partial_scalar_mul(pow_real_fn(n, t), beta_pow_seq(m, t), m.suc)
        pow_real_fn(n, t) * partial[Real](beta_pow_seq(m, t), m.suc) =
            partial[Real](mul_fn(pow_real_fn(n, t), beta_pow_seq(m, t)), m.suc)
        forall(k: Nat) {
            mul_fn(pow_real_fn(n, t), beta_pow_seq(m, t), k) =
                pow_real_fn(n, t) * beta_pow_seq(m, t, k)
            beta_pow_seq(m, t, k) =
                alternating_sign[Real](k) * from_nat[Real](m.binom(k)) * pow_real_fn(k, t)
            pow_real_fn(n, t) * (alternating_sign[Real](k) * from_nat[Real](m.binom(k)) * pow_real_fn(k, t)) =
                alternating_sign[Real](k) * from_nat[Real](m.binom(k)) *
                    (pow_real_fn(n, t) * pow_real_fn(k, t))
            pow_real_fn(n, t) * pow_real_fn(k, t) = t.pow(n) * t.pow(k)
            pow_add(t, n, k)
            t.pow(n + k) = t.pow(n) * t.pow(k)
            pow_real_fn(n + k, t) = t.pow(n + k)
            pow_real_fn(n, t) * pow_real_fn(k, t) = pow_real_fn(n + k, t)
            mul_fn(pow_real_fn(n, t), beta_pow_seq(m, t), k) =
                alternating_sign[Real](k) * from_nat[Real](m.binom(k)) * pow_real_fn(n + k, t)
            beta_deriv_term_fn(n, m, k, t) =
                alternating_sign[Real](k) * from_nat[Real](m.binom(k)) * pow_real_fn(n + k, t)
            mul_fn(pow_real_fn(n, t), beta_pow_seq(m, t), k) = beta_deriv_term_fn(n, m, k, t)
        }
        partial_pointwise_eq(mul_fn(pow_real_fn(n, t), beta_pow_seq(m, t)), beta_deriv_seq(n, m, t), m.suc)
        partial[Real](mul_fn(pow_real_fn(n, t), beta_pow_seq(m, t)), m.suc) =
            partial[Real](beta_deriv_seq(n, m, t), m.suc)
        pow_real_fn(n, t) * partial[Real](beta_pow_seq(m, t), m.suc) =
            partial[Real](beta_deriv_seq(n, m, t), m.suc)
        beta_alternating_pow(m)
        partial[Real](beta_pow_seq(m, t), m.suc) = pow_real_fn(m, unit_minus(t))
        pow_real_fn(n, t) * pow_real_fn(m, unit_minus(t)) =
            partial[Real](beta_deriv_seq(n, m, t), m.suc)
        beta_fn(n, m, t) = pow_real_fn(n, t) * pow_real_fn(m, unit_minus(t))
        beta_anti_deriv(n, m, t) = beta_fn(n, m, t)
    }
    function_extensionality(beta_anti_deriv(n, m), beta_fn(n, m))
}

/// The derivative of the Beta antiderivative is the Beta integrand.
theorem beta_anti_derivative_is_beta_fn(n: Nat, m: Nat) {
    is_derivative_fn(beta_anti(n, m), beta_fn(n, m))
} by {
    beta_anti_derivative(n, m)
    is_derivative_fn(beta_anti(n, m), beta_anti_deriv(n, m))
    beta_anti_deriv_eq_beta_fn(n, m)
    beta_anti_deriv(n, m) = beta_fn(n, m)
    function_eq_transport_predicate_rev[Real, Real](
        function(h: Real -> Real) { is_derivative_fn(beta_anti(n, m), h) },
        beta_anti_deriv(n, m), beta_fn(n, m))
    is_derivative_fn(beta_anti(n, m), beta_fn(n, m))
}

// ---------------------------------------------------------------------------
// Bounds and Lipschitz constants for the Beta integrand on [0, 1]
// ---------------------------------------------------------------------------

/// The complement of a unit-interval point lies in the unit interval.
lemma unit_minus_in_unit(t: Real) {
    interval_contains(Real.0, Real.1, t) implies interval_contains(Real.0, Real.1, unit_minus(t))
} by {
    if interval_contains(Real.0, Real.1, t) {
        interval_contains_left(Real.0, Real.1, t)
        Real.0 <= t
        interval_contains_right(Real.0, Real.1, t)
        t <= Real.1
        unit_minus(t) = affine_real(-Real.1, Real.1, t)
        affine_real(-Real.1, Real.1, t) = -Real.1 * t + Real.1
        unit_minus(t) = -t + Real.1
        Real.1 - t = -t + Real.1
        unit_minus(t) = Real.1 - t
        neg_lte_flip(Real.0, t)
        -t <= -Real.0
        -Real.0 = Real.0
        -t <= Real.0
        add_le_add_right[Real](-t, Real.0, Real.1)
        -t + Real.1 <= Real.0 + Real.1
        Real.0 + Real.1 = Real.1
        -t + Real.1 <= Real.1
        unit_minus(t) <= Real.1
        Real.0 <= unit_minus(t) and unit_minus(t) <= Real.1
        interval_contains(Real.0, Real.1, unit_minus(t)) =
            Real.0 <= unit_minus(t) and unit_minus(t) <= Real.1
        interval_contains(Real.0, Real.1, unit_minus(t))
    }
}

/// The Beta integrand is nonnegative and at most one on [0, 1].
theorem beta_fn_unit_bounds(n: Nat, m: Nat) {
    forall(t: Real) {
        interval_contains(Real.0, Real.1, t) implies
        Real.0 <= beta_fn(n, m, t) and beta_fn(n, m, t) <= Real.1
    }
} by {
    forall(t: Real) {
        if interval_contains(Real.0, Real.1, t) {
            pow_real_fn_nonneg_unit(n)
            pow_real_fn_nonneg_unit(m)
            pow_real_fn_le_one_unit(n)
            pow_real_fn_le_one_unit(m)
            unit_minus_in_unit(t)
            interval_contains(Real.0, Real.1, unit_minus(t))
            pow_real_fn_nonneg_unit(m)
            Real.0 <= pow_real_fn(m, unit_minus(t))
            pow_real_fn_le_one_unit(m)
            pow_real_fn(m, unit_minus(t)) <= Real.1
            pow_real_fn_nonneg_unit(n)
            Real.0 <= pow_real_fn(n, t)
            beta_fn(n, m, t) = pow_real_fn(n, t) * pow_real_fn(m, unit_minus(t))
            mul_le_mul_of_nonneg_right[Real](Real.0, pow_real_fn(m, unit_minus(t)), pow_real_fn(n, t))
            Real.0 * pow_real_fn(n, t) <= pow_real_fn(m, unit_minus(t)) * pow_real_fn(n, t)
            Real.0 * pow_real_fn(n, t) = Real.0
            pow_real_fn(m, unit_minus(t)) * pow_real_fn(n, t) =
                pow_real_fn(n, t) * pow_real_fn(m, unit_minus(t))
            Real.0 <= beta_fn(n, m, t)
            mul_le_mul_of_nonneg_right[Real](pow_real_fn(m, unit_minus(t)), Real.1, pow_real_fn(n, t))
            pow_real_fn(m, unit_minus(t)) * pow_real_fn(n, t) <= Real.1 * pow_real_fn(n, t)
            Real.1 * pow_real_fn(n, t) = pow_real_fn(n, t)
            pow_real_fn(m, unit_minus(t)) * pow_real_fn(n, t) <= pow_real_fn(n, t)
            pow_real_fn(n, t) <= Real.1
            lte_trans[Real](pow_real_fn(m, unit_minus(t)) * pow_real_fn(n, t), pow_real_fn(n, t), Real.1)
            pow_real_fn(m, unit_minus(t)) * pow_real_fn(n, t) <= Real.1
            beta_fn(n, m, t) = pow_real_fn(n, t) * pow_real_fn(m, unit_minus(t))
            pow_real_fn(m, unit_minus(t)) * pow_real_fn(n, t) =
                beta_fn(n, m, t)
            beta_fn(n, m, t) <= Real.1
            Real.0 <= beta_fn(n, m, t) and beta_fn(n, m, t) <= Real.1
        }
    }
}

/// The product of two Lipschitz functions on an interval is Lipschitz with the
/// product constant.
theorem lipschitz_mul_on(
    f: Real -> Real, g: Real -> Real, a: Real, b: Real, lf: Real, lg: Real, bf: Real, bg: Real
) {
    Real.0 <= lf and Real.0 <= lg and Real.0 <= bf and Real.0 <= bg and
    (forall(u: Real, v: Real) {
        interval_contains(a, b, u) and interval_contains(a, b, v) implies
        (f(u) - f(v)).abs <= lf * (u - v).abs
    }) and
    (forall(u: Real, v: Real) {
        interval_contains(a, b, u) and interval_contains(a, b, v) implies
        (g(u) - g(v)).abs <= lg * (u - v).abs
    }) and
    (forall(t: Real) { interval_contains(a, b, t) implies f(t).abs <= bf }) and
    (forall(t: Real) { interval_contains(a, b, t) implies g(t).abs <= bg })
    implies
    (forall(u: Real, v: Real) {
        interval_contains(a, b, u) and interval_contains(a, b, v) implies
        (f(u) * g(u) - f(v) * g(v)).abs <= (bf * lg + bg * lf) * (u - v).abs
    })
} by {
    if Real.0 <= lf and Real.0 <= lg and Real.0 <= bf and Real.0 <= bg and
       (forall(u: Real, v: Real) {
           interval_contains(a, b, u) and interval_contains(a, b, v) implies
           (f(u) - f(v)).abs <= lf * (u - v).abs
       }) and
       (forall(u: Real, v: Real) {
           interval_contains(a, b, u) and interval_contains(a, b, v) implies
           (g(u) - g(v)).abs <= lg * (u - v).abs
       }) and
       (forall(t: Real) { interval_contains(a, b, t) implies f(t).abs <= bf }) and
       (forall(t: Real) { interval_contains(a, b, t) implies g(t).abs <= bg }) {
        forall(u: Real, v: Real) {
            if interval_contains(a, b, u) and interval_contains(a, b, v) {
                f(u) * g(u) - f(v) * g(v) =
                    f(u) * (g(u) - g(v)) + g(v) * (f(u) - f(v))
                triangle_ineq(f(u) * (g(u) - g(v)), g(v) * (f(u) - f(v)))
                (f(u) * (g(u) - g(v)) + g(v) * (f(u) - f(v))).abs <= (f(u) * (g(u) - g(v))).abs + (g(v) * (f(u) - f(v))).abs
                mul_abs(f(u), g(u) - g(v))
                (f(u) * (g(u) - g(v))).abs = f(u).abs * (g(u) - g(v)).abs
                mul_abs(g(v), f(u) - f(v))
                (g(v) * (f(u) - f(v))).abs = g(v).abs * (f(u) - f(v)).abs
                abs_gte_zero(u - v)
                Real.0 <= (u - v).abs
                abs_gte_zero(g(u) - g(v))
                Real.0 <= (g(u) - g(v)).abs
                abs_gte_zero(f(u) - f(v))
                Real.0 <= (f(u) - f(v)).abs
                (g(u) - g(v)).abs <= lg * (u - v).abs
                (f(u) - f(v)).abs <= lf * (u - v).abs
                f(u).abs <= bf
                g(v).abs <= bg
                mul_le_mul_of_nonneg_right[Real](f(u).abs, bf, (g(u) - g(v)).abs)
                f(u).abs * (g(u) - g(v)).abs <= bf * (g(u) - g(v)).abs
                mul_le_mul_of_nonneg_right[Real]((g(u) - g(v)).abs, lg * (u - v).abs, bf)
                (g(u) - g(v)).abs * bf <= (lg * (u - v).abs) * bf
                (g(u) - g(v)).abs * bf = bf * (g(u) - g(v)).abs
                (lg * (u - v).abs) * bf = bf * (lg * (u - v).abs)
                lte_trans[Real](bf * (g(u) - g(v)).abs, (lg * (u - v).abs) * bf, bf * (lg * (u - v).abs))
                bf * (g(u) - g(v)).abs <= bf * (lg * (u - v).abs)
                lte_trans[Real](f(u).abs * (g(u) - g(v)).abs, bf * (g(u) - g(v)).abs, bf * (lg * (u - v).abs))
                f(u).abs * (g(u) - g(v)).abs <= bf * (lg * (u - v).abs)
                bf * (lg * (u - v).abs) = (bf * lg) * (u - v).abs
                mul_le_mul_of_nonneg_right[Real](g(v).abs, bg, (f(u) - f(v)).abs)
                g(v).abs * (f(u) - f(v)).abs <= bg * (f(u) - f(v)).abs
                mul_le_mul_of_nonneg_right[Real]((f(u) - f(v)).abs, lf * (u - v).abs, bg)
                (f(u) - f(v)).abs * bg <= (lf * (u - v).abs) * bg
                (f(u) - f(v)).abs * bg = bg * (f(u) - f(v)).abs
                (lf * (u - v).abs) * bg = bg * (lf * (u - v).abs)
                lte_trans[Real](bg * (f(u) - f(v)).abs, (lf * (u - v).abs) * bg, bg * (lf * (u - v).abs))
                bg * (f(u) - f(v)).abs <= bg * (lf * (u - v).abs)
                lte_trans[Real](g(v).abs * (f(u) - f(v)).abs, bg * (f(u) - f(v)).abs, bg * (lf * (u - v).abs))
                g(v).abs * (f(u) - f(v)).abs <= bg * (lf * (u - v).abs)
                bg * (lf * (u - v).abs) = (bg * lf) * (u - v).abs
                lte_trans[Real](
                    f(u).abs * (g(u) - g(v)).abs,
                    bf * (lg * (u - v).abs),
                    (bf * lg) * (u - v).abs)
                f(u).abs * (g(u) - g(v)).abs <= (bf * lg) * (u - v).abs
                lte_trans[Real](
                    g(v).abs * (f(u) - f(v)).abs,
                    bg * (lf * (u - v).abs),
                    (bg * lf) * (u - v).abs)
                g(v).abs * (f(u) - f(v)).abs <= (bg * lf) * (u - v).abs
                add_le_add(
                    f(u).abs * (g(u) - g(v)).abs, (bf * lg) * (u - v).abs,
                    g(v).abs * (f(u) - f(v)).abs, (bg * lf) * (u - v).abs)
                f(u).abs * (g(u) - g(v)).abs + g(v).abs * (f(u) - f(v)).abs <= (bf * lg) * (u - v).abs + (bg * lf) * (u - v).abs
                mul_distrib_left((bf * lg), (bg * lf), (u - v).abs)
                ((bf * lg) + (bg * lf)) * (u - v).abs = (bf * lg) * (u - v).abs + (bg * lf) * (u - v).abs
                lte_trans[Real](
                    f(u).abs * (g(u) - g(v)).abs + g(v).abs * (f(u) - f(v)).abs,
                    (bf * lg) * (u - v).abs + (bg * lf) * (u - v).abs,
                    ((bf * lg) + (bg * lf)) * (u - v).abs)
                f(u).abs * (g(u) - g(v)).abs + g(v).abs * (f(u) - f(v)).abs <= ((bf * lg) + (bg * lf)) * (u - v).abs
                lte_trans[Real](
                    (f(u) * g(u) - f(v) * g(v)).abs,
                    f(u).abs * (g(u) - g(v)).abs + g(v).abs * (f(u) - f(v)).abs,
                    ((bf * lg) + (bg * lf)) * (u - v).abs)
                (f(u) * g(u) - f(v) * g(v)).abs <= ((bf * lg) + (bg * lf)) * (u - v).abs
            }
        }
    }
}

/// The complement of the unit interval is 1-Lipschitz.
lemma unit_minus_lipschitz_unit {
    forall(u: Real, v: Real) {
        interval_contains(Real.0, Real.1, u) and interval_contains(Real.0, Real.1, v) implies
        (unit_minus(u) - unit_minus(v)).abs <= Real.1 * (u - v).abs
    }
} by {
    forall(u: Real, v: Real) {
        if interval_contains(Real.0, Real.1, u) and interval_contains(Real.0, Real.1, v) {
            unit_minus(u) = Real.1 - u
            unit_minus(v) = Real.1 - v
            unit_minus(u) - unit_minus(v) = (Real.1 - u) - (Real.1 - v)
            (Real.1 - u) - (Real.1 - v) = v - u
            (unit_minus(u) - unit_minus(v)).abs = (v - u).abs
            (v - u).abs = (u - v).abs
            Real.1 * (u - v).abs = (u - v).abs
            (unit_minus(u) - unit_minus(v)).abs <= Real.1 * (u - v).abs
        }
    }
}

/// The second factor of the Beta integrand is Lipschitz.
lemma beta_fn_factor_lipschitz(n: Nat, m: Nat) {
    forall(u: Real, v: Real) {
        interval_contains(Real.0, Real.1, u) and interval_contains(Real.0, Real.1, v) implies
        (pow_real_fn(m, unit_minus(u)) - pow_real_fn(m, unit_minus(v))).abs <= from_nat[Real](m) * (u - v).abs
    }
} by {
    forall(u: Real, v: Real) {
        if interval_contains(Real.0, Real.1, u) and interval_contains(Real.0, Real.1, v) {
            unit_minus_in_unit(u)
            interval_contains(Real.0, Real.1, unit_minus(u))
            unit_minus_in_unit(v)
            interval_contains(Real.0, Real.1, unit_minus(v))
            monomial_lipschitz_unit(m)
            (pow_real_fn(m, unit_minus(u)) - pow_real_fn(m, unit_minus(v))).abs <= from_nat[Real](m) * (unit_minus(u) - unit_minus(v)).abs
            unit_minus_lipschitz_unit
            (unit_minus(u) - unit_minus(v)).abs <= Real.1 * (u - v).abs
            Real.1 * (u - v).abs = (u - v).abs
            from_nat_real_nonneg(m)
            Real.0 <= from_nat[Real](m)
            mul_le_mul_of_nonneg_right[Real](
                (unit_minus(u) - unit_minus(v)).abs, (u - v).abs, from_nat[Real](m))
            (unit_minus(u) - unit_minus(v)).abs * from_nat[Real](m) <= (u - v).abs * from_nat[Real](m)
            (unit_minus(u) - unit_minus(v)).abs * from_nat[Real](m) =
                from_nat[Real](m) * (unit_minus(u) - unit_minus(v)).abs
            (u - v).abs * from_nat[Real](m) = from_nat[Real](m) * (u - v).abs
            from_nat[Real](m) * (unit_minus(u) - unit_minus(v)).abs <= from_nat[Real](m) * (u - v).abs
            lte_trans[Real](
                (pow_real_fn(m, unit_minus(u)) - pow_real_fn(m, unit_minus(v))).abs,
                from_nat[Real](m) * (unit_minus(u) - unit_minus(v)).abs,
                from_nat[Real](m) * (u - v).abs)
            (pow_real_fn(m, unit_minus(u)) - pow_real_fn(m, unit_minus(v))).abs <= from_nat[Real](m) * (u - v).abs
        }
    }
}

/// Both factors of the Beta integrand are n- and m-Lipschitz respectively.
lemma beta_fn_factors_lipschitz(n: Nat, m: Nat) {
    forall(u: Real, v: Real) {
        interval_contains(Real.0, Real.1, u) and interval_contains(Real.0, Real.1, v) implies
        (pow_real_fn(n, u) - pow_real_fn(n, v)).abs <= from_nat[Real](n) * (u - v).abs
    }
} by {
    forall(u: Real, v: Real) {
        if interval_contains(Real.0, Real.1, u) and interval_contains(Real.0, Real.1, v) {
            monomial_lipschitz_unit(n)
            (pow_real_fn(n, u) - pow_real_fn(n, v)).abs <= from_nat[Real](n) * (u - v).abs
        }
    }
}

/// The first Beta factor is at most one in absolute value on [0, 1].
lemma beta_fn_bound_abs_first(n: Nat, m: Nat) {
    forall(t: Real) { interval_contains(Real.0, Real.1, t) implies pow_real_fn(n, t).abs <= Real.1 }
} by {
    forall(t: Real) {
        if interval_contains(Real.0, Real.1, t) {
            pow_real_fn_nonneg_unit(n)
            Real.0 <= pow_real_fn(n, t)
            abs_of_nonneg(pow_real_fn(n, t))
            pow_real_fn(n, t).abs = pow_real_fn(n, t)
            pow_real_fn_le_one_unit(n)
            pow_real_fn(n, t) <= Real.1
            pow_real_fn(n, t).abs <= Real.1
        }
    }
    forall(t: Real) {
        if interval_contains(Real.0, Real.1, t) {
            unit_minus_in_unit(t)
            interval_contains(Real.0, Real.1, unit_minus(t))
            pow_real_fn_nonneg_unit(m)
            Real.0 <= pow_real_fn(m, unit_minus(t))
            abs_of_nonneg(pow_real_fn(m, unit_minus(t)))
            pow_real_fn(m, unit_minus(t)).abs = pow_real_fn(m, unit_minus(t))
            pow_real_fn_le_one_unit(m)
            pow_real_fn(m, unit_minus(t)) <= Real.1
            pow_real_fn(m, unit_minus(t)).abs <= Real.1
        }
    }
}

/// The second Beta factor is at most one in absolute value on [0, 1].
lemma beta_fn_bound_abs_second(n: Nat, m: Nat) {
    forall(t: Real) {
        interval_contains(Real.0, Real.1, t) implies pow_real_fn(m, unit_minus(t)).abs <= Real.1
    }
} by {
    forall(t: Real) {
        if interval_contains(Real.0, Real.1, t) {
            unit_minus_in_unit(t)
            interval_contains(Real.0, Real.1, unit_minus(t))
            pow_real_fn_nonneg_unit(m)
            Real.0 <= pow_real_fn(m, unit_minus(t))
            abs_of_nonneg(pow_real_fn(m, unit_minus(t)))
            pow_real_fn(m, unit_minus(t)).abs = pow_real_fn(m, unit_minus(t))
            pow_real_fn_le_one_unit(m)
            pow_real_fn(m, unit_minus(t)) <= Real.1
            pow_real_fn(m, unit_minus(t)).abs <= Real.1
        }
    }
}

/// The Lipschitz bound on the product of the Beta factors, instantiated.
lemma lipschitz_mul_on_lipschitz(n: Nat, m: Nat, u: Real, v: Real) {
    interval_contains(Real.0, Real.1, u) and interval_contains(Real.0, Real.1, v) implies
    (pow_real_fn(n, u) * pow_real_fn(m, unit_minus(u)) -
        pow_real_fn(n, v) * pow_real_fn(m, unit_minus(v))).abs <= (Real.1 * from_nat[Real](m) + Real.1 * from_nat[Real](n)) * (u - v).abs
} by {
    if interval_contains(Real.0, Real.1, u) and interval_contains(Real.0, Real.1, v) {
        monomial_lipschitz_unit(n)
        monomial_lipschitz_unit(m)
        unit_minus_lipschitz_unit
        from_nat_real_nonneg(n)
        from_nat[Real](n) >= Real.0
        Real.0 <= from_nat[Real](n)
        from_nat_real_nonneg(m)
        from_nat[Real](m) >= Real.0
        Real.0 <= from_nat[Real](m)
        Real.0 <= Real.1
        zero_is_smaller_than_one[Real]
        Real.0 < Real.1
        lt_imp_lte[Real](Real.0, Real.1)
        beta_fn_factors_lipschitz(n, m)
        (forall(u0: Real, v0: Real) {
            interval_contains(Real.0, Real.1, u0) and interval_contains(Real.0, Real.1, v0) implies
            (pow_real_fn(n, u0) - pow_real_fn(n, v0)).abs <= from_nat[Real](n) * (u0 - v0).abs
        })
        eq_true_intro(forall(u0: Real, v0: Real) {
            interval_contains(Real.0, Real.1, u0) and interval_contains(Real.0, Real.1, v0) implies
            (pow_real_fn(n, u0) - pow_real_fn(n, v0)).abs <= from_nat[Real](n) * (u0 - v0).abs
        })
        (forall(u0: Real, v0: Real) {
            interval_contains(Real.0, Real.1, u0) and interval_contains(Real.0, Real.1, v0) implies
            (pow_real_fn(n, u0) - pow_real_fn(n, v0)).abs <= from_nat[Real](n) * (u0 - v0).abs
        }) = true
        beta_fn_factor_lipschitz(n, m)
        (forall(u0: Real, v0: Real) {
            interval_contains(Real.0, Real.1, u0) and interval_contains(Real.0, Real.1, v0) implies
            (pow_real_fn(m, unit_minus(u0)) - pow_real_fn(m, unit_minus(v0))).abs <= from_nat[Real](m) * (u0 - v0).abs
        })
        eq_true_intro(forall(u0: Real, v0: Real) {
            interval_contains(Real.0, Real.1, u0) and interval_contains(Real.0, Real.1, v0) implies
            (pow_real_fn(m, unit_minus(u0)) - pow_real_fn(m, unit_minus(v0))).abs <= from_nat[Real](m) * (u0 - v0).abs
        })
        (forall(u0: Real, v0: Real) {
            interval_contains(Real.0, Real.1, u0) and interval_contains(Real.0, Real.1, v0) implies
            (pow_real_fn(m, unit_minus(u0)) - pow_real_fn(m, unit_minus(v0))).abs <= from_nat[Real](m) * (u0 - v0).abs
        }) = true
        beta_fn_bound_abs_first(n, m)
        (forall(t: Real) {
            interval_contains(Real.0, Real.1, t) implies pow_real_fn(n, t).abs <= Real.1
        })
        eq_true_intro(forall(t: Real) {
            interval_contains(Real.0, Real.1, t) implies pow_real_fn(n, t).abs <= Real.1
        })
        (forall(t: Real) {
            interval_contains(Real.0, Real.1, t) implies pow_real_fn(n, t).abs <= Real.1
        }) = true
        beta_fn_bound_abs_second(n, m)
        (forall(t: Real) {
            interval_contains(Real.0, Real.1, t) implies pow_real_fn(m, unit_minus(t)).abs <= Real.1
        })
        eq_true_intro(forall(t: Real) {
            interval_contains(Real.0, Real.1, t) implies pow_real_fn(m, unit_minus(t)).abs <= Real.1
        })
        (forall(t: Real) {
            interval_contains(Real.0, Real.1, t) implies pow_real_fn(m, unit_minus(t)).abs <= Real.1
        }) = true
        Real.0 <= from_nat[Real](n) and Real.0 <= from_nat[Real](m) and Real.0 <= Real.1 and Real.0 <= Real.1 and
            (forall(u0: Real, v0: Real) {
                interval_contains(Real.0, Real.1, u0) and interval_contains(Real.0, Real.1, v0) implies
                (pow_real_fn(n, u0) - pow_real_fn(n, v0)).abs <= from_nat[Real](n) * (u0 - v0).abs
            }) and
            (forall(u0: Real, v0: Real) {
                interval_contains(Real.0, Real.1, u0) and interval_contains(Real.0, Real.1, v0) implies
                (pow_real_fn(m, unit_minus(u0)) - pow_real_fn(m, unit_minus(v0))).abs <= from_nat[Real](m) * (u0 - v0).abs
            }) and
            (forall(t: Real) {
                interval_contains(Real.0, Real.1, t) implies pow_real_fn(n, t).abs <= Real.1
            }) and
            (forall(t: Real) {
                interval_contains(Real.0, Real.1, t) implies pow_real_fn(m, unit_minus(t)).abs <= Real.1
            })
        lipschitz_mul_on(pow_real_fn(n),
            function(x: Real) { pow_real_fn(m, unit_minus(x)) },
            Real.0, Real.1, from_nat[Real](n), from_nat[Real](m), Real.1, Real.1)
        (pow_real_fn(n, u) * pow_real_fn(m, unit_minus(u)) -
            pow_real_fn(n, v) * pow_real_fn(m, unit_minus(v))).abs <= (Real.1 * from_nat[Real](m) + Real.1 * from_nat[Real](n)) * (u - v).abs
    }
}

/// The Beta integrand is (n+m)-Lipschitz on [0, 1].
theorem beta_fn_lipschitz_unit(n: Nat, m: Nat) {
    forall(u: Real, v: Real) {
        interval_contains(Real.0, Real.1, u) and interval_contains(Real.0, Real.1, v) implies
        (beta_fn(n, m, u) - beta_fn(n, m, v)).abs <= from_nat[Real](n + m) * (u - v).abs
    }
} by {
    forall(u: Real, v: Real) {
        if interval_contains(Real.0, Real.1, u) and interval_contains(Real.0, Real.1, v) {
            lipschitz_mul_on_lipschitz(n, m, u, v)
            (pow_real_fn(n, u) * pow_real_fn(m, unit_minus(u)) -
                pow_real_fn(n, v) * pow_real_fn(m, unit_minus(v))).abs <= (Real.1 * from_nat[Real](m) + Real.1 * from_nat[Real](n)) * (u - v).abs
            beta_fn(n, m, u) = pow_real_fn(n, u) * pow_real_fn(m, unit_minus(u))
            beta_fn(n, m, v) = pow_real_fn(n, v) * pow_real_fn(m, unit_minus(v))
            (beta_fn(n, m, u) - beta_fn(n, m, v)).abs <= (Real.1 * from_nat[Real](m) + Real.1 * from_nat[Real](n)) * (u - v).abs
            Real.1 * from_nat[Real](m) = from_nat[Real](m)
            Real.1 * from_nat[Real](n) = from_nat[Real](n)
            from_nat_add[Real](m, n)
            from_nat[Real](m + n) = from_nat[Real](m) + from_nat[Real](n)
            add_comm(m, n)
            m + n = n + m
            Real.1 * from_nat[Real](m) + Real.1 * from_nat[Real](n) =
                from_nat[Real](n + m)
            (beta_fn(n, m, u) - beta_fn(n, m, v)).abs <= from_nat[Real](n + m) * (u - v).abs
        }
    }
}






// ---------------------------------------------------------------------------
// Integrability and the value of the Beta function
// ---------------------------------------------------------------------------

/// The Beta integrand is integrable on [0, 1].
theorem beta_integrable(n: Nat, m: Nat) {
    is_integrable(beta_fn(n, m), Real.0, Real.1)
} by {
    beta_anti_continuous(n, m)
    continuous(beta_anti(n, m))
    beta_anti_derivative_is_beta_fn(n, m)
    is_derivative_fn(beta_anti(n, m), beta_fn(n, m))
    beta_fn_lipschitz_unit(n, m)
    beta_fn_unit_bounds(n, m)
    Real.0 <= Real.1
    zero_is_smaller_than_one[Real]
    Real.0 < Real.1
    lt_imp_lte[Real](Real.0, Real.1)
    from_nat_real_nonneg(n + m)
    Real.0 <= from_nat[Real](n + m)
    eq_true_intro(continuous(beta_anti(n, m)))
    (continuous(beta_anti(n, m))) = true
    eq_true_intro(is_derivative_fn(beta_anti(n, m), beta_fn(n, m)))
    (is_derivative_fn(beta_anti(n, m), beta_fn(n, m))) = true
    eq_true_intro(forall(u: Real, v: Real) {
        interval_contains(Real.0, Real.1, u) and interval_contains(Real.0, Real.1, v) implies
        (beta_fn(n, m, u) - beta_fn(n, m, v)).abs <= from_nat[Real](n + m) * (u - v).abs
    })
    (forall(u: Real, v: Real) {
        interval_contains(Real.0, Real.1, u) and interval_contains(Real.0, Real.1, v) implies
        (beta_fn(n, m, u) - beta_fn(n, m, v)).abs <= from_nat[Real](n + m) * (u - v).abs
    }) = true
    eq_true_intro(forall(t: Real) {
        interval_contains(Real.0, Real.1, t) implies Real.0 <= beta_fn(n, m, t)
    })
    (forall(t: Real) {
        interval_contains(Real.0, Real.1, t) implies Real.0 <= beta_fn(n, m, t)
    }) = true
    eq_true_intro(forall(t: Real) {
        interval_contains(Real.0, Real.1, t) implies beta_fn(n, m, t) <= Real.1
    })
    (forall(t: Real) {
        interval_contains(Real.0, Real.1, t) implies beta_fn(n, m, t) <= Real.1
    }) = true
    Real.0 <= Real.1 and Real.0 <= from_nat[Real](n + m) and
        continuous(beta_anti(n, m)) and
        is_derivative_fn(beta_anti(n, m), beta_fn(n, m)) and
        (forall(u: Real, v: Real) {
            interval_contains(Real.0, Real.1, u) and interval_contains(Real.0, Real.1, v) implies
            (beta_fn(n, m, u) - beta_fn(n, m, v)).abs <= from_nat[Real](n + m) * (u - v).abs
        }) and
        (forall(t: Real) {
            interval_contains(Real.0, Real.1, t) implies Real.0 <= beta_fn(n, m, t)
        }) and
        (forall(t: Real) {
            interval_contains(Real.0, Real.1, t) implies beta_fn(n, m, t) <= Real.1
        })
    fn_integrable_gen(beta_fn(n, m), beta_anti(n, m), Real.0, Real.1,
        from_nat[Real](n + m), Real.0, Real.1)
    is_integrable(beta_fn(n, m), Real.0, Real.1)
}

/// The Beta antiderivative at 1 is the alternating binomial sum.
lemma beta_anti_value_one(n: Nat, m: Nat) {
    beta_anti(n, m, Real.1) = beta_sum(n, m)
} by {
    beta_anti(n, m, Real.1) = partial[Real](beta_anti_seq(n, m, Real.1), m.suc)
    forall(k: Nat) {
        beta_anti_seq(n, m, Real.1, k) = beta_anti_term_fn(n, m, k, Real.1)
        beta_anti_term_fn(n, m, k, Real.1) = beta_anti_coeff(n, m, k) * pow_real_fn(n.suc + k, Real.1)
        pow_real_fn(n.suc + k, Real.1) = Real.1.pow(n.suc + k)
        one_pow[Real](n.suc + k)
        Real.1.pow(n.suc + k) = Real.1
        beta_anti_term_fn(n, m, k, Real.1) = beta_anti_coeff(n, m, k)
        beta_anti_coeff(n, m, k) =
            alternating_sign[Real](k) * from_nat[Real](m.binom(k)) / from_nat[Real](n.suc + k)
        beta_sum_term(n, m, k) =
            alternating_sign[Real](k) * from_nat[Real](m.binom(k)) / from_nat[Real](n.suc + k)
        beta_anti_seq(n, m, Real.1, k) = beta_sum_term(n, m, k)
    }
    partial_pointwise_eq(beta_anti_seq(n, m, Real.1), beta_sum_term(n, m), m.suc)
    partial[Real](beta_anti_seq(n, m, Real.1), m.suc) = partial[Real](beta_sum_term(n, m), m.suc)
    beta_sum(n, m) = partial[Real](beta_sum_term(n, m), m.suc)
    beta_anti(n, m, Real.1) = beta_sum(n, m)
}

/// The partial sum of the constant-zero sequence is zero.
lemma partial_zero_seq(n: Nat) {
    partial[Real](constant[Nat, Real](Real.0), n) = Real.0
} by {
    define p(x: Nat) -> Bool {
        partial[Real](constant[Nat, Real](Real.0), x) = Real.0
    }
    forall(t: Real) {
        partial[Real](constant[Nat, Real](Real.0), Nat.0) = Real.0
        partial_zero(constant[Nat, Real](Real.0))
    }
    p(Nat.0)
    forall(x: Nat) {
        if p(x) {
            partial[Real](constant[Nat, Real](Real.0), x.suc) =
                partial[Real](constant[Nat, Real](Real.0), x) + constant[Nat, Real](Real.0, x)
            partial_split_last(constant[Nat, Real](Real.0), x)
            partial[Real](constant[Nat, Real](Real.0), x) = Real.0
            constant[Nat, Real](Real.0, x) = Real.0
            partial[Real](constant[Nat, Real](Real.0), x.suc) = Real.0
            p(x.suc)
        }
    }
    p(Nat.0) and forall(x: Nat) { p(x) implies p(x.suc) }
    alt_induction(p)
    forall(x: Nat) { p(x) }
    p(n)
}

/// The Beta antiderivative at 0 is zero.
lemma beta_anti_value_zero(n: Nat, m: Nat) {
    beta_anti(n, m, Real.0) = Real.0
} by {
    beta_anti(n, m, Real.0) = partial[Real](beta_anti_seq(n, m, Real.0), m.suc)
    forall(k: Nat) {
        beta_anti_seq(n, m, Real.0, k) = beta_anti_term_fn(n, m, k, Real.0)
        beta_anti_term_fn(n, m, k, Real.0) = beta_anti_coeff(n, m, k) * pow_real_fn(n.suc + k, Real.0)
        pow_real_fn(n.suc + k, Real.0) = Real.0.pow(n.suc + k)
        Nat.1 <= n.suc
        n.suc + Nat.0 = n.suc
        lte_add_left(n.suc, Nat.0, k)
        n.suc + Nat.0 <= n.suc + k
        n.suc <= n.suc + k
        lte_trans[Nat](Nat.1, n.suc, n.suc + k)
        Nat.1 <= n.suc + k
        n.suc + k >= Nat.1
        zero_pow_pos(n.suc + k)
        Real.0.pow(n.suc + k) = Real.0
        beta_anti_coeff(n, m, k) * Real.0 = Real.0
        beta_anti_term_fn(n, m, k, Real.0) = beta_anti_coeff(n, m, k) * pow_real_fn(n.suc + k, Real.0)
        beta_anti_term_fn(n, m, k, Real.0) = beta_anti_coeff(n, m, k) * Real.0
        beta_anti_term_fn(n, m, k, Real.0) = Real.0
        beta_anti_seq(n, m, Real.0, k) = Real.0
        constant[Nat, Real](Real.0, k) = Real.0
        beta_anti_seq(n, m, Real.0, k) = constant[Nat, Real](Real.0, k)
    }
    partial_pointwise_eq(beta_anti_seq(n, m, Real.0), constant[Nat, Real](Real.0), m.suc)
    partial[Real](beta_anti_seq(n, m, Real.0), m.suc) =
        partial[Real](constant[Nat, Real](Real.0), m.suc)
    partial_zero_seq(m.suc)
    partial[Real](constant[Nat, Real](Real.0), m.suc) = Real.0
    beta_anti(n, m, Real.0) = Real.0
}


/// The Beta function equals the alternating binomial sum.
theorem beta_value(n: Nat, m: Nat) {
    beta(n, m) = beta_sum(n, m)
} by {
    beta_integrable(n, m)
    is_integrable(beta_fn(n, m), Real.0, Real.1)
    beta_anti_continuous(n, m)
    continuous(beta_anti(n, m))
    beta_anti_derivative_is_beta_fn(n, m)
    is_derivative_fn(beta_anti(n, m), beta_fn(n, m))
    beta_fn_unit_bounds(n, m)
    Real.0 <= Real.1
    zero_is_smaller_than_one[Real]
    Real.0 < Real.1
    lt_imp_lte[Real](Real.0, Real.1)
    eq_true_intro(continuous(beta_anti(n, m)))
    (continuous(beta_anti(n, m))) = true
    eq_true_intro(is_derivative_fn(beta_anti(n, m), beta_fn(n, m)))
    (is_derivative_fn(beta_anti(n, m), beta_fn(n, m))) = true
    eq_true_intro(forall(t: Real) {
        interval_contains(Real.0, Real.1, t) implies Real.0 <= beta_fn(n, m, t)
    })
    (forall(t: Real) {
        interval_contains(Real.0, Real.1, t) implies Real.0 <= beta_fn(n, m, t)
    }) = true
    eq_true_intro(forall(t: Real) {
        interval_contains(Real.0, Real.1, t) implies beta_fn(n, m, t) <= Real.1
    })
    (forall(t: Real) {
        interval_contains(Real.0, Real.1, t) implies beta_fn(n, m, t) <= Real.1
    }) = true
    Real.0 <= Real.1 and is_integrable(beta_fn(n, m), Real.0, Real.1) and
        continuous(beta_anti(n, m)) and
        is_derivative_fn(beta_anti(n, m), beta_fn(n, m)) and
        (forall(t: Real) {
            interval_contains(Real.0, Real.1, t) implies Real.0 <= beta_fn(n, m, t)
        }) and
        (forall(t: Real) {
            interval_contains(Real.0, Real.1, t) implies beta_fn(n, m, t) <= Real.1
        })
    ftc2_general(beta_fn(n, m), beta_anti(n, m), Real.0, Real.1, Real.0, Real.1)
    integral(beta_fn(n, m), Real.0, Real.1) = beta_anti(n, m, Real.1) - beta_anti(n, m, Real.0)
    beta_anti_value_one(n, m)
    beta_anti(n, m, Real.1) = beta_sum(n, m)
    beta_anti_value_zero(n, m)
    beta_anti(n, m, Real.0) = Real.0
    beta_sum(n, m) - Real.0 = beta_sum(n, m)
    integral(beta_fn(n, m), Real.0, Real.1) = beta_sum(n, m)
    beta(n, m) = integral(beta_fn(n, m), Real.0, Real.1)
    beta(n, m) = beta_sum(n, m)
}

// ---------------------------------------------------------------------------
// The alternating binomial sum in closed form
// ---------------------------------------------------------------------------

/// The pointwise identity behind the Beta sum recurrence: the shifted
/// alternating term is minus the n+1 term plus the shifted n term.
lemma beta_sum_recurrence_term(n: Nat, m: Nat, k: Nat) {
    beta_sum_term(n, m.suc, k.suc) = -beta_sum_term(n + Nat.1, m, k) + beta_sum_term(n, m, k.suc)
} by {
    beta_sum_term(n, m.suc, k.suc) =
        alternating_sign[Real](k.suc) * from_nat[Real](m.suc.binom(k.suc)) / from_nat[Real](n.suc + k.suc)
    pascal_suc_unbounded(m, k)
    m.suc.binom(k.suc) = m.binom(k) + m.binom(k.suc)
    alternating_sign_suc[Real](k)
    alternating_sign[Real](k.suc) = -alternating_sign[Real](k)
    from_nat[Real](m.suc.binom(k.suc)) = from_nat[Real](m.binom(k) + m.binom(k.suc))
    from_nat_add[Real](m.binom(k), m.binom(k.suc))
    from_nat[Real](m.binom(k) + m.binom(k.suc)) =
        from_nat[Real](m.binom(k)) + from_nat[Real](m.binom(k.suc))
    beta_sum_term(n, m.suc, k.suc) =
        (-alternating_sign[Real](k)) *
            (from_nat[Real](m.binom(k)) + from_nat[Real](m.binom(k.suc))) / from_nat[Real](n.suc + k.suc)
    n.suc + k.suc = n.suc.suc + k
    from_nat_suc_pos_real(n + k.suc)
    from_nat[Real]((n + k.suc).suc) > Real.0
    from_nat[Real]((n + k.suc).suc) != Real.0
    (n + k.suc).suc = n.suc + k.suc
    from_nat[Real](n.suc + k.suc) != Real.0
    mul_distrib_left(-alternating_sign[Real](k),
        from_nat[Real](m.binom(k)), from_nat[Real](m.binom(k.suc)))
    (-alternating_sign[Real](k)) * (from_nat[Real](m.binom(k)) + from_nat[Real](m.binom(k.suc))) =
        (-alternating_sign[Real](k)) * from_nat[Real](m.binom(k)) +
        (-alternating_sign[Real](k)) * from_nat[Real](m.binom(k.suc))
    alternating_sign[Real](k.suc) = -alternating_sign[Real](k)
    beta_sum_term(n, m.suc, k.suc) =
        ((-alternating_sign[Real](k)) * from_nat[Real](m.binom(k)) +
            alternating_sign[Real](k.suc) * from_nat[Real](m.binom(k.suc))) / from_nat[Real](n.suc + k.suc)
    beta_sum_term(n + Nat.1, m, k) =
        alternating_sign[Real](k) * from_nat[Real](m.binom(k)) / from_nat[Real]((n + Nat.1).suc + k)
    (n + Nat.1).suc + k = n.suc.suc + k
    n.suc.suc + k = n.suc + k.suc
    from_nat[Real]((n + Nat.1).suc + k) = from_nat[Real](n.suc + k.suc)
    beta_sum_term(n + Nat.1, m, k) =
        alternating_sign[Real](k) * from_nat[Real](m.binom(k)) / from_nat[Real](n.suc + k.suc)
    beta_sum_term(n, m, k.suc) =
        alternating_sign[Real](k.suc) * from_nat[Real](m.binom(k.suc)) / from_nat[Real](n.suc + k.suc)
    div_add_same_denom(Real.1,
        -alternating_sign[Real](k) * from_nat[Real](m.binom(k)),
        alternating_sign[Real](k.suc) * from_nat[Real](m.binom(k.suc)),
        from_nat[Real](n.suc + k.suc))
    ((-alternating_sign[Real](k)) * from_nat[Real](m.binom(k))) / from_nat[Real](n.suc + k.suc) +
        (alternating_sign[Real](k.suc) * from_nat[Real](m.binom(k.suc))) / from_nat[Real](n.suc + k.suc) =
        ((-alternating_sign[Real](k)) * from_nat[Real](m.binom(k)) +
            alternating_sign[Real](k.suc) * from_nat[Real](m.binom(k.suc))) / from_nat[Real](n.suc + k.suc)
    -beta_sum_term(n + Nat.1, m, k) + beta_sum_term(n, m, k.suc) =
        -alternating_sign[Real](k) * from_nat[Real](m.binom(k)) / from_nat[Real](n.suc + k.suc) +
            alternating_sign[Real](k.suc) * from_nat[Real](m.binom(k.suc)) / from_nat[Real](n.suc + k.suc)
    -beta_sum_term(n + Nat.1, m, k) + beta_sum_term(n, m, k.suc) =
        beta_sum_term(n, m.suc, k.suc)
}

/// The top alternating term of the n-sum vanishes: C(m, m+1) = 0.
lemma beta_sum_recurrence_top_zero(n: Nat, m: Nat) {
    beta_sum_term(n, m, m.suc) = Real.0
} by {
    beta_sum_term(n, m, m.suc) =
        alternating_sign[Real](m.suc) * from_nat[Real](m.binom(m.suc)) / from_nat[Real](n.suc + m.suc)
    m < m.suc
    m.binom(m.suc) = Nat.0
    from_nat_zero[Real]
    from_nat[Real](Nat.0) = Real.0
    from_nat[Real](m.binom(m.suc)) = Real.0
    alternating_sign[Real](m.suc) * Real.0 = Real.0
    alternating_sign[Real](m.suc) * Real.0 / from_nat[Real](n.suc + m.suc) = Real.0
    beta_sum_term(n, m, m.suc) = Real.0
}

/// The alternating binomial sum splits as the difference of the sums for m
/// and for n+1: S(n, m+1) = S(n, m) - S(n+1, m).
theorem beta_sum_recurrence(n: Nat, m: Nat) {
    beta_sum(n, m.suc) = beta_sum(n, m) - beta_sum(n + Nat.1, m)
} by {
    define p(k: Nat) -> Real { beta_sum_term(n, m.suc, k) }
    define a(k: Nat) -> Real { beta_sum_term(n, m, k) }
    define u(k: Nat) -> Real { beta_sum_term(n + Nat.1, m, k) }
    define neg_u(k: Nat) -> Real { -u(k) }
    define shift_a(k: Nat) -> Real { a(k.suc) }
    define sum_ua(k: Nat) -> Real { -u(k) + a(k.suc) }
    p(Nat.0) = beta_sum_term(n, m.suc, Nat.0)
    a(Nat.0) = beta_sum_term(n, m, Nat.0)
    beta_sum_term(n, m.suc, Nat.0) =
        alternating_sign[Real](Nat.0) * from_nat[Real](m.suc.binom(Nat.0)) / from_nat[Real](n.suc)
    alternating_sign_zero[Real]
    alternating_sign[Real](Nat.0) = Real.1
    choose_zero(m.suc)
    m.suc.binom(Nat.0) = Nat.1
    from_nat_one[Real]
    from_nat[Real](Nat.1) = Real.1
    from_nat[Real](m.suc.binom(Nat.0)) = Real.1
    beta_sum_term(n, m.suc, Nat.0) = Real.1 * Real.1 / from_nat[Real](n.suc)
    Real.1 * Real.1 = Real.1
    Real.1 * Real.1 / from_nat[Real](n.suc) = Real.1 / from_nat[Real](n.suc)
    beta_sum_term(n, m.suc, Nat.0) = Real.1 / from_nat[Real](n.suc)
    beta_sum_term(n, m, Nat.0) =
        alternating_sign[Real](Nat.0) * from_nat[Real](m.binom(Nat.0)) / from_nat[Real](n.suc)
    choose_zero(m)
    m.binom(Nat.0) = Nat.1
    from_nat[Real](m.binom(Nat.0)) = Real.1
    beta_sum_term(n, m, Nat.0) = Real.1 * Real.1 / from_nat[Real](n.suc)
    beta_sum_term(n, m, Nat.0) = Real.1 / from_nat[Real](n.suc)
    p(Nat.0) = a(Nat.0)
    forall(k: Nat) {
        beta_sum_recurrence_term(n, m, k)
        p(k.suc) = -u(k) + a(k.suc)
    }
    partial_shift_suc(p, m.suc)
    p(Nat.0) + partial[Real](compose(p, Nat.suc), m.suc) = partial[Real](p, m.suc.suc)
    beta_sum(n, m.suc) = partial[Real](p, m.suc.suc)
    forall(k: Nat) {
        compose(p, Nat.suc, k) = p(k.suc)
        p(k.suc) = -u(k) + a(k.suc)
        sum_ua(k) = -u(k) + a(k.suc)
        compose(p, Nat.suc, k) = sum_ua(k)
    }
    partial_pointwise_eq(compose(p, Nat.suc), sum_ua, m.suc)
    partial[Real](compose(p, Nat.suc), m.suc) = partial[Real](sum_ua, m.suc)
    forall(k: Nat) {
        sum_ua(k) = -u(k) + a(k.suc)
        neg_u(k) = -u(k)
        shift_a(k) = a(k.suc)
        add_fn(neg_u, shift_a, k) = neg_u(k) + shift_a(k)
        sum_ua(k) = add_fn(neg_u, shift_a, k)
    }
    partial_pointwise_eq(sum_ua, add_fn(neg_u, shift_a), m.suc)
    partial[Real](sum_ua, m.suc) = partial[Real](add_fn(neg_u, shift_a), m.suc)
    partial_add(neg_u, shift_a, m.suc)
    partial[Real](neg_u, m.suc) + partial[Real](shift_a, m.suc) =
        partial[Real](add_fn(neg_u, shift_a), m.suc)
    forall(k: Nat) {
        neg_u(k) = -u(k)
        mul_fn(-Real.1, u, k) = -Real.1 * u(k)
        mul_neg_left(-Real.1, u(k))
        -Real.1 * u(k) = -u(k)
        neg_u(k) = mul_fn(-Real.1, u, k)
    }
    partial_pointwise_eq(neg_u, mul_fn(-Real.1, u), m.suc)
    partial[Real](neg_u, m.suc) = partial[Real](mul_fn(-Real.1, u), m.suc)
    partial_scalar_mul(-Real.1, u, m.suc)
    -Real.1 * partial[Real](u, m.suc) = partial[Real](mul_fn(-Real.1, u), m.suc)
    -Real.1 * partial[Real](u, m.suc) = partial[Real](neg_u, m.suc)
    partial_shift_suc(a, m.suc)
    a(Nat.0) + partial[Real](compose(a, Nat.suc), m.suc) = partial[Real](a, m.suc.suc)
    partial[Real](compose(a, Nat.suc), m.suc) =
        partial[Real](a, m.suc.suc) - a(Nat.0)
    forall(k: Nat) {
        shift_a(k) = a(k.suc)
        compose(a, Nat.suc, k) = a(k.suc)
        shift_a(k) = compose(a, Nat.suc, k)
    }
    partial_pointwise_eq(shift_a, compose(a, Nat.suc), m.suc)
    partial[Real](shift_a, m.suc) = partial[Real](compose(a, Nat.suc), m.suc)
    partial[Real](shift_a, m.suc) = partial[Real](a, m.suc.suc) - a(Nat.0)
    beta_sum(n, m) = partial[Real](a, m.suc)
    partial_split_last(a, m.suc)
    partial[Real](a, m.suc.suc) = partial[Real](a, m.suc) + a(m.suc)
    beta_sum_recurrence_top_zero(n, m)
    a(m.suc) = Real.0
    partial[Real](a, m.suc.suc) = beta_sum(n, m)
    partial[Real](shift_a, m.suc) = beta_sum(n, m) - a(Nat.0)
    beta_sum(n + Nat.1, m) = partial[Real](u, m.suc)
    partial[Real](compose(p, Nat.suc), m.suc) = partial[Real](sum_ua, m.suc)
    partial[Real](sum_ua, m.suc) = partial[Real](add_fn(neg_u, shift_a), m.suc)
    partial[Real](neg_u, m.suc) + partial[Real](shift_a, m.suc) =
        partial[Real](compose(p, Nat.suc), m.suc)
    -Real.1 * partial[Real](u, m.suc) + partial[Real](shift_a, m.suc) =
        partial[Real](compose(p, Nat.suc), m.suc)
    -beta_sum(n + Nat.1, m) + (beta_sum(n, m) - a(Nat.0)) =
        partial[Real](compose(p, Nat.suc), m.suc)
    partial[Real](p, m.suc.suc) = p(Nat.0) + (-beta_sum(n + Nat.1, m) + (beta_sum(n, m) - a(Nat.0)))
    p(Nat.0) = a(Nat.0)
    beta_sum(n, m) - a(Nat.0) = beta_sum(n, m) + -a(Nat.0)
    -beta_sum(n + Nat.1, m) + (beta_sum(n, m) + -a(Nat.0)) =
        -beta_sum(n + Nat.1, m) + beta_sum(n, m) + -a(Nat.0)
    -beta_sum(n + Nat.1, m) + (beta_sum(n, m) - a(Nat.0)) =
        -beta_sum(n + Nat.1, m) + beta_sum(n, m) - a(Nat.0)
    a(Nat.0) + (-beta_sum(n + Nat.1, m) + beta_sum(n, m) - a(Nat.0)) =
        a(Nat.0) + (-beta_sum(n + Nat.1, m) + beta_sum(n, m)) - a(Nat.0)
    a(Nat.0) + (-beta_sum(n + Nat.1, m) + beta_sum(n, m)) - a(Nat.0) =
        a(Nat.0) + (-beta_sum(n + Nat.1, m) + beta_sum(n, m)) + -a(Nat.0)
    sub_cancels(-beta_sum(n + Nat.1, m) + beta_sum(n, m), a(Nat.0))
    (-beta_sum(n + Nat.1, m) + beta_sum(n, m)) + a(Nat.0) - a(Nat.0) =
        -beta_sum(n + Nat.1, m) + beta_sum(n, m)
    a(Nat.0) + (-beta_sum(n + Nat.1, m) + beta_sum(n, m)) =
        (-beta_sum(n + Nat.1, m) + beta_sum(n, m)) + a(Nat.0)
    a(Nat.0) + (-beta_sum(n + Nat.1, m) + beta_sum(n, m)) + -a(Nat.0) =
        -beta_sum(n + Nat.1, m) + beta_sum(n, m)
    a(Nat.0) + (-beta_sum(n + Nat.1, m) + (beta_sum(n, m) - a(Nat.0))) =
        -beta_sum(n + Nat.1, m) + beta_sum(n, m)
    -beta_sum(n + Nat.1, m) + beta_sum(n, m) = beta_sum(n, m) - beta_sum(n + Nat.1, m)
    a(Nat.0) + (-beta_sum(n + Nat.1, m) + (beta_sum(n, m) - a(Nat.0))) =
        beta_sum(n, m) - beta_sum(n + Nat.1, m)
    beta_sum(n, m.suc) = beta_sum(n, m) - beta_sum(n + Nat.1, m)
}

/// The closed form at m = 0: S(n, 0) = 1/(n+1).
theorem beta_sum_base(n: Nat) {
    beta_sum(n, Nat.0) = Real.1 / from_nat[Real](n.suc)
} by {
    beta_sum(n, Nat.0) = partial[Real](beta_sum_term(n, Nat.0), Nat.0.suc)
    partial_one(beta_sum_term(n, Nat.0))
    partial[Real](beta_sum_term(n, Nat.0), Nat.1) = beta_sum_term(n, Nat.0, Nat.0)
    beta_sum_term(n, Nat.0, Nat.0) =
        alternating_sign[Real](Nat.0) * from_nat[Real](Nat.0.binom(Nat.0)) / from_nat[Real](n.suc)
    alternating_sign_zero[Real]
    alternating_sign[Real](Nat.0) = Real.1
    choose_zero(Nat.0)
    Nat.0.binom(Nat.0) = Nat.1
    from_nat_one[Real]
    from_nat[Real](Nat.1) = Real.1
    from_nat[Real](Nat.0.binom(Nat.0)) = Real.1
    Real.1 * from_nat[Real](Nat.0.binom(Nat.0)) = Real.1
    Real.1 * from_nat[Real](Nat.0.binom(Nat.0)) / from_nat[Real](n.suc) = Real.1 / from_nat[Real](n.suc)
    beta_sum(n, Nat.0) = Real.1 / from_nat[Real](n.suc)
}

/// The fraction algebra of the closed-form recurrence: with c = p + q and
/// c, d nonzero, a·b/d - (p·a)·b/(c·d) = a·(q·b)/(c·d).
lemma beta_closed_algebra(a: Real, b: Real, d: Real, c: Real, p: Real, q: Real) {
    c != Real.0 and d != Real.0 and c = p + q
    implies
    a * b / d - (p * a) * b / (c * d) = a * (q * b) / (c * d)
} by {
    if c != Real.0 and d != Real.0 and c = p + q {
        c * d != Real.0
        div_cancel_common(a * b, c, d)
        ((a * b) * c) / (d * c) = (a * b) / d
        (a * b) * c = a * b * c
        d * c = c * d
        a * b * c / (c * d) = a * b / d
        div_sub_same_denom(Real.1, Real.1, a * b * c, (p * a) * b, c * d)
        a * b * c / (c * d) - ((p * a) * b) / (c * d) =
            (a * b * c - (p * a) * b) / (c * d)
        (p * a) * b = p * (a * b)
        c * (a * b) = a * b * c
        mul_sub_distrib_left(c, p, a * b)
        (c - p) * (a * b) = c * (a * b) - p * (a * b)
        (c - p) * (a * b) = a * b * c - (p * a) * b
        a * b * c - (p * a) * b = a * b * (c - p)
        p + q - p = q
        c = p + q
        (p + q) - p = q
        c - p = q
        a * b * (c - p) = a * b * q
        (a * b * c - (p * a) * b) / (c * d) = (a * b * q) / (c * d)
        (a * b) * q = a * (q * b)
        a * b * q = (a * b) * q
        a * b * q = a * (q * b)
        a * b / d - (p * a) * b / (c * d) = a * (q * b) / (c * d)
    }
}

/// The successor sum identity behind the closed-form recurrence:
/// (n+1) + (m+1) = n + m + 2.
lemma beta_closed_algebra_c_sum(n: Nat, m: Nat) {
    from_nat[Real](n + m + Nat.1.suc) = from_nat[Real](n + Nat.1) + from_nat[Real](m.suc)
} by {
    from_nat_add[Real](n + Nat.1, m.suc)
    from_nat[Real]((n + Nat.1) + m.suc) = from_nat[Real](n + Nat.1) + from_nat[Real](m.suc)
    add_assoc(n, Nat.1, m.suc)
    n + (Nat.1 + m.suc) = (n + Nat.1) + m.suc
    add_comm(Nat.1, m.suc)
    Nat.1 + m.suc = m.suc + Nat.1
    add_assoc(m.suc, n, Nat.1)
    (m.suc + n) + Nat.1 = m.suc + (n + Nat.1)
    m.suc + (n + Nat.1) = (m.suc + n) + Nat.1
    m.suc + Nat.1 = m.suc.suc
    m.suc.suc = m + Nat.1.suc
    n + (m.suc + Nat.1) = n + (m + Nat.1.suc)
    add_assoc(n, m, Nat.1.suc)
    n + (m + Nat.1.suc) = (n + m) + Nat.1.suc
    n + (m.suc + Nat.1) = n + m + Nat.1.suc
    Nat.1 + m.suc = m.suc + Nat.1
    n + (Nat.1 + m.suc) = n + m + Nat.1.suc
    (n + Nat.1) + m.suc = n + m + Nat.1.suc
    from_nat[Real](n + m + Nat.1.suc) = from_nat[Real](n + Nat.1) + from_nat[Real](m.suc)
}

/// The closed form satisfies the recurrence: n!m!/(n+m+1)! minus
/// (n+1)!m!/(n+m+2)! equals n!(m+1)!/(n+m+2)!.
lemma beta_closed_recurrence(n: Nat, m: Nat) {
    beta_closed(n, m) - beta_closed(n + Nat.1, m) = beta_closed(n, m.suc)
} by {
    beta_closed(n, m) =
        from_nat[Real](n.factorial) * from_nat[Real](m.factorial) / from_nat[Real]((n + m + Nat.1).factorial)
    beta_closed(n + Nat.1, m) =
        from_nat[Real]((n + Nat.1).factorial) * from_nat[Real](m.factorial) /
            from_nat[Real]((n + Nat.1 + m + Nat.1).factorial)
    factorial_step(n)
    (n + Nat.1).factorial = (n + Nat.1) * n.factorial
    from_nat_mul[Real](n + Nat.1, n.factorial)
    from_nat[Real]((n + Nat.1) * n.factorial) = from_nat[Real](n + Nat.1) * from_nat[Real](n.factorial)
    from_nat[Real]((n + Nat.1).factorial) = from_nat[Real](n + Nat.1) * from_nat[Real](n.factorial)
    factorial_step(n + m + Nat.1)
    (n + m + Nat.1.suc).factorial = (n + m + Nat.1.suc) * (n + m + Nat.1).factorial
    from_nat_mul[Real](n + m + Nat.1.suc, (n + m + Nat.1).factorial)
    from_nat[Real]((n + m + Nat.1.suc) * (n + m + Nat.1).factorial) =
        from_nat[Real](n + m + Nat.1.suc) * from_nat[Real]((n + m + Nat.1).factorial)
    (n + Nat.1 + m + Nat.1).factorial = (n + m + Nat.1.suc).factorial
    from_nat[Real]((n + Nat.1 + m + Nat.1).factorial) =
        from_nat[Real](n + m + Nat.1.suc) * from_nat[Real]((n + m + Nat.1).factorial)
    factorial_step(m)
    m.suc.factorial = m.suc * m.factorial
    from_nat_mul[Real](m.suc, m.factorial)
    from_nat[Real](m.suc.factorial) = from_nat[Real](m.suc) * from_nat[Real](m.factorial)
    (n + m.suc + Nat.1).factorial = (n + m + Nat.1.suc).factorial
    beta_closed(n, m.suc) =
        from_nat[Real](n.factorial) * from_nat[Real](m.suc.factorial) /
            from_nat[Real]((n + m.suc + Nat.1).factorial)
    from_nat[Real](n.factorial) * (from_nat[Real](m.suc) * from_nat[Real](m.factorial)) /
        (from_nat[Real](n + m + Nat.1.suc) * from_nat[Real]((n + m + Nat.1).factorial)) =
        beta_closed(n, m.suc)
    from_nat_suc_pos_real(n + m + Nat.1)
    from_nat[Real]((n + m + Nat.1).suc) > Real.0
    (n + m + Nat.1).suc = n + m + Nat.1.suc
    from_nat[Real](n + m + Nat.1.suc) != Real.0
    from_nat_suc_pos_real(n + m + Nat.1)
    from_nat[Real]((n + m + Nat.1).suc) > Real.0
    from_nat[Real]((n + m + Nat.1).factorial) != Real.0
    beta_closed_algebra_c_sum(n, m)
    from_nat[Real](n + m + Nat.1.suc) = from_nat[Real](n + Nat.1) + from_nat[Real](m.suc)
    beta_closed_algebra(
        from_nat[Real](n.factorial),
        from_nat[Real](m.factorial),
        from_nat[Real]((n + m + Nat.1).factorial),
        from_nat[Real](n + m + Nat.1.suc),
        from_nat[Real](n + Nat.1),
        from_nat[Real](m.suc))
    from_nat[Real](n.factorial) * from_nat[Real](m.factorial) / from_nat[Real]((n + m + Nat.1).factorial) -
        (from_nat[Real](n + Nat.1) * from_nat[Real](n.factorial)) * from_nat[Real](m.factorial) /
            (from_nat[Real](n + m + Nat.1.suc) * from_nat[Real]((n + m + Nat.1).factorial)) =
        from_nat[Real](n.factorial) * (from_nat[Real](m.suc) * from_nat[Real](m.factorial)) /
            (from_nat[Real](n + m + Nat.1.suc) * from_nat[Real]((n + m + Nat.1).factorial))
    from_nat[Real]((n + Nat.1).factorial) * from_nat[Real](m.factorial) /
        from_nat[Real]((n + Nat.1 + m + Nat.1).factorial) =
        (from_nat[Real](n + Nat.1) * from_nat[Real](n.factorial)) * from_nat[Real](m.factorial) /
            (from_nat[Real](n + m + Nat.1.suc) * from_nat[Real]((n + m + Nat.1).factorial))
    beta_closed(n + Nat.1, m) =
        (from_nat[Real](n + Nat.1) * from_nat[Real](n.factorial)) * from_nat[Real](m.factorial) /
            (from_nat[Real](n + m + Nat.1.suc) * from_nat[Real]((n + m + Nat.1).factorial))
    beta_closed(n, m) - beta_closed(n + Nat.1, m) =
        from_nat[Real](n.factorial) * (from_nat[Real](m.suc) * from_nat[Real](m.factorial)) /
            (from_nat[Real](n + m + Nat.1.suc) * from_nat[Real]((n + m + Nat.1).factorial))
    beta_closed(n, m) - beta_closed(n + Nat.1, m) = beta_closed(n, m.suc)
}


/// The alternating binomial sum in closed form:
/// Σ_{j=0}^{m} (-1)^j · C(m,j)/(n+1+j) = n!·m!/(n+m+1)!.
theorem beta_sum_value(n: Nat, m: Nat) {
    beta_sum(n, m) = beta_closed(n, m)
} by {
    define p(x: Nat) -> Bool {
        forall(n0: Nat) { beta_sum(n0, x) = beta_closed(n0, x) }
    }

    // Base case: m = 0.
    forall(n0: Nat) {
        beta_sum_base(n0)
        beta_sum(n0, Nat.0) = Real.1 / from_nat[Real](n0.suc)
        beta_closed(n0, Nat.0) =
            from_nat[Real](n0.factorial) * from_nat[Real](Nat.0.factorial) /
                from_nat[Real]((n0 + Nat.0 + Nat.1).factorial)
        factorial_zero
        Nat.0.factorial = Nat.1
        from_nat_one[Real]
        from_nat[Real](Nat.1) = Real.1
        from_nat[Real](Nat.0.factorial) = Real.1
        n0 + Nat.0 + Nat.1 = n0.suc
        factorial_step(n0)
        (n0.suc).factorial = n0.suc * n0.factorial
        from_nat_mul[Real](n0.suc, n0.factorial)
        from_nat[Real](n0.suc.factorial) = from_nat[Real](n0.suc) * from_nat[Real](n0.factorial)
        (n0 + Nat.0 + Nat.1).factorial = n0.suc.factorial
        from_nat[Real]((n0 + Nat.0 + Nat.1).factorial) =
            from_nat[Real](n0.suc) * from_nat[Real](n0.factorial)
        from_nat_suc_pos_real(n0)
        from_nat[Real](n0.suc) > Real.0
        from_nat[Real](n0.suc) != Real.0
        from_nat_suc_pos_real(n0)
        from_nat[Real](n0.suc) > Real.0
        from_nat[Real](n0.factorial) != Real.0
        div_cancel_common(Real.1, from_nat[Real](n0.factorial), from_nat[Real](n0.suc))
        (Real.1 * from_nat[Real](n0.factorial)) /
            (from_nat[Real](n0.suc) * from_nat[Real](n0.factorial)) = Real.1 / from_nat[Real](n0.suc)
        from_nat[Real](n0.factorial) * Real.1 =
            Real.1 * from_nat[Real](n0.factorial)
        from_nat[Real](n0.factorial) * from_nat[Real](Nat.0.factorial) /
            from_nat[Real]((n0 + Nat.0 + Nat.1).factorial) =
            Real.1 / from_nat[Real](n0.suc)
        beta_closed(n0, Nat.0) =
            from_nat[Real](n0.factorial) * from_nat[Real](Nat.0.factorial) /
                from_nat[Real]((n0 + Nat.0 + Nat.1).factorial)
        beta_closed(n0, Nat.0) = Real.1 / from_nat[Real](n0.suc)
        beta_sum(n0, Nat.0) = beta_closed(n0, Nat.0)
    }
    p(Nat.0)

    // Inductive step.
    forall(x: Nat) {
        if p(x) {
            forall(n0: Nat) {
                beta_sum_recurrence(n0, x)
                beta_sum(n0, x.suc) = beta_sum(n0, x) - beta_sum(n0 + Nat.1, x)
                p(x)
                beta_sum(n0, x) = beta_closed(n0, x)
                beta_sum(n0 + Nat.1, x) = beta_closed(n0 + Nat.1, x)
                beta_closed_recurrence(n0, x)
                beta_closed(n0, x) - beta_closed(n0 + Nat.1, x) = beta_closed(n0, x.suc)
                beta_sum(n0, x.suc) = beta_closed(n0, x.suc)
            }
            p(x.suc)
        }
    }

    p(Nat.0) and forall(x: Nat) { p(x) implies p(x.suc) }
    alt_induction(p)
    forall(x: Nat) { p(x) }
    p(m)
    forall(n0: Nat) { beta_sum(n0, m) = beta_closed(n0, m) }
    beta_sum(n, m) = beta_closed(n, m)
}

// ---------------------------------------------------------------------------
// Consequences: closed form, boundary values, symmetry
// ---------------------------------------------------------------------------

/// The Beta function in closed form: beta(n, m) = n!·m!/(n+m+1)!.
theorem beta_closed_value(n: Nat, m: Nat) {
    beta(n, m) = beta_closed(n, m)
} by {
    beta_value(n, m)
    beta(n, m) = beta_sum(n, m)
    beta_sum_value(n, m)
    beta_sum(n, m) = beta_closed(n, m)
    beta(n, m) = beta_closed(n, m)
}

/// The boundary value beta(n, 0) = 1/(n+1).
theorem beta_n_zero(n: Nat) {
    beta(n, Nat.0) = Real.1 / from_nat[Real](n.suc)
} by {
    beta_closed_value(n, Nat.0)
    beta(n, Nat.0) = beta_closed(n, Nat.0)
    beta_closed(n, Nat.0) =
        from_nat[Real](n.factorial) * from_nat[Real](Nat.0.factorial) /
            from_nat[Real]((n + Nat.0 + Nat.1).factorial)
    factorial_zero
    Nat.0.factorial = Nat.1
    from_nat_one[Real]
    from_nat[Real](Nat.1) = Real.1
    from_nat[Real](Nat.0.factorial) = Real.1
    n + Nat.0 + Nat.1 = n.suc
    factorial_step(n)
    (n.suc).factorial = n.suc * n.factorial
    from_nat_mul[Real](n.suc, n.factorial)
    from_nat[Real](n.suc.factorial) = from_nat[Real](n.suc) * from_nat[Real](n.factorial)
    (n + Nat.0 + Nat.1).factorial = n.suc.factorial
    from_nat[Real]((n + Nat.0 + Nat.1).factorial) =
        from_nat[Real](n.suc) * from_nat[Real](n.factorial)
    from_nat_suc_pos_real(n)
    from_nat[Real](n.suc) > Real.0
    from_nat[Real](n.suc) != Real.0
    from_nat_suc_pos_real(n)
    from_nat[Real](n.suc) > Real.0
    from_nat[Real](n.factorial) != Real.0
    div_cancel_common(Real.1, from_nat[Real](n.factorial), from_nat[Real](n.suc))
    (Real.1 * from_nat[Real](n.factorial)) /
        (from_nat[Real](n.suc) * from_nat[Real](n.factorial)) = Real.1 / from_nat[Real](n.suc)
    from_nat[Real](n.factorial) * Real.1 = Real.1 * from_nat[Real](n.factorial)
    from_nat[Real](n.factorial) * from_nat[Real](Nat.0.factorial) /
        from_nat[Real]((n + Nat.0 + Nat.1).factorial) = Real.1 / from_nat[Real](n.suc)
    beta_closed(n, Nat.0) = Real.1 / from_nat[Real](n.suc)
    beta(n, Nat.0) = Real.1 / from_nat[Real](n.suc)
}

/// The Beta function is symmetric: beta(n, m) = beta(m, n).
theorem beta_sym(n: Nat, m: Nat) {
    beta(n, m) = beta(m, n)
} by {
    beta_closed_value(n, m)
    beta(n, m) = beta_closed(n, m)
    beta_closed_value(m, n)
    beta(m, n) = beta_closed(m, n)
    real_mul_comm(from_nat[Real](n.factorial), from_nat[Real](m.factorial))
    from_nat[Real](n.factorial) * from_nat[Real](m.factorial) =
        from_nat[Real](m.factorial) * from_nat[Real](n.factorial)
    add_comm(n, m)
    n + m = m + n
    (n + m + Nat.1).factorial = (m + n + Nat.1).factorial
    from_nat[Real]((n + m + Nat.1).factorial) = from_nat[Real]((m + n + Nat.1).factorial)
    beta_closed(n, m) = beta_closed(m, n)
    beta(n, m) = beta(m, n)
}

/// The successor-sum identity behind the shifted closed form:
/// (n-1) + (m-1) + 1 = n + m - 1 for n, m >= 1.
lemma beta_shift_sum(n: Nat, m: Nat) {
    Nat.1 <= n and Nat.1 <= m implies (n - Nat.1) + (m - Nat.1) + Nat.1 = n + m - Nat.1
} by {
    if Nat.1 <= n and Nat.1 <= m {
        add_sub(n, Nat.1)
        n - Nat.1 + Nat.1 = n
        add_sub(m, Nat.1)
        m - Nat.1 + Nat.1 = m
        add_assoc(n - Nat.1, m - Nat.1, Nat.1)
        (n - Nat.1) + (m - Nat.1) + Nat.1 = (n - Nat.1) + ((m - Nat.1) + Nat.1)
        (n - Nat.1) + ((m - Nat.1) + Nat.1) = (n - Nat.1) + m
        add_assoc(n - Nat.1, m, Nat.1)
        ((n - Nat.1) + m) + Nat.1 = (n - Nat.1) + (m + Nat.1)
        add_comm(m, Nat.1)
        m + Nat.1 = Nat.1 + m
        (n - Nat.1) + (m + Nat.1) = (n - Nat.1) + (Nat.1 + m)
        add_assoc(n - Nat.1, Nat.1, m)
        (n - Nat.1) + (Nat.1 + m) = ((n - Nat.1) + Nat.1) + m
        ((n - Nat.1) + Nat.1) + m = n + m
        ((n - Nat.1) + m) + Nat.1 = n + m
        add_sub(n + m, Nat.1)
        (n + m) - Nat.1 + Nat.1 = n + m
        add_cancels_right((n - Nat.1) + m, (n + m) - Nat.1, Nat.1)
        (n - Nat.1) + m = n + m - Nat.1
        (n - Nat.1) + (m - Nat.1) + Nat.1 = (n - Nat.1) + m
        (n - Nat.1) + (m - Nat.1) + Nat.1 = n + m - Nat.1
    }
}

/// The shifted closed form: the integral of t^(n-1)·(1-t)^(m-1) over [0, 1]
/// is (n-1)!·(m-1)!/(n+m-1)! for n, m >= 1.
theorem beta_shifted(n: Nat, m: Nat) {
    Nat.1 <= n and Nat.1 <= m implies
    beta(n - Nat.1, m - Nat.1) =
        from_nat[Real]((n - Nat.1).factorial) * from_nat[Real]((m - Nat.1).factorial) /
            from_nat[Real]((n + m - Nat.1).factorial)
} by {
    if Nat.1 <= n and Nat.1 <= m {
        beta_closed_value(n - Nat.1, m - Nat.1)
        beta(n - Nat.1, m - Nat.1) = beta_closed(n - Nat.1, m - Nat.1)
        beta_closed(n - Nat.1, m - Nat.1) =
            from_nat[Real]((n - Nat.1).factorial) * from_nat[Real]((m - Nat.1).factorial) /
                from_nat[Real](((n - Nat.1) + (m - Nat.1) + Nat.1).factorial)
        beta_shift_sum(n, m)
        (n - Nat.1) + (m - Nat.1) + Nat.1 = n + m - Nat.1
        ((n - Nat.1) + (m - Nat.1) + Nat.1).factorial = (n + m - Nat.1).factorial
        from_nat[Real](((n - Nat.1) + (m - Nat.1) + Nat.1).factorial) =
            from_nat[Real]((n + m - Nat.1).factorial)
        beta_closed(n - Nat.1, m - Nat.1) =
            from_nat[Real]((n - Nat.1).factorial) * from_nat[Real]((m - Nat.1).factorial) /
                from_nat[Real]((n + m - Nat.1).factorial)
        beta(n - Nat.1, m - Nat.1) =
            from_nat[Real]((n - Nat.1).factorial) * from_nat[Real]((m - Nat.1).factorial) /
                from_nat[Real]((n + m - Nat.1).factorial)
    }
}
