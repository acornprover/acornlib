/// Integrability criteria for the Darboux integral.
///
/// This file develops the machinery around the Darboux criterion: a bounded
/// function on [a, b] whose upper and lower Darboux sums can be made
/// arbitrarily close over a single partition is integrable.  The classical
/// applications (continuous functions, functions with finitely many
/// discontinuities, and the Riemann-Lebesgue criterion) are stated at the
/// end; each needs an ingredient that the library does not yet provide:
///   - the extreme value theorem (continuity on [a, b] implies boundedness),
///   - Heine-Cantor (continuity on [a, b] implies uniform continuity on
///     [a, b]; the library only has global uniform continuity),
///   - the cross-partition comparison of Darboux sums (any lower sum is at
///     most any upper sum), which needs the common-refinement machinery of
///     partitions.
/// The Darboux criterion itself is proved below in a conditional form: it is
/// stated with the cross-partition comparison as an explicit hypothesis, so
/// that the entire remaining structure of the classical proof (boundedness
/// of the sum sets, existence of the supremum and infimum, and the
/// eps-condition forcing them together) is verified here.  Uniform
/// continuity is shown to imply the eps-condition, and as a complete
/// instance of the criterion a bounded function that is constant on the open
/// interval (a, b) is proved integrable, with integral c * (b - a).

from nat import Nat, from_nat, lt_imp_lte_suc, pos_of_ne_zero, lte_suc_suc, lte_add_right, add_sub
from rat import Rat
from order import lte_antisymm, lte_trans, lt_imp_lte, lt_of_lte_of_lt, lt_of_lt_of_lte, lt_of_lte_of_ne, not_lte_imp_gt, not_lt_self
from real.real_field import Real, mul_inverse, div_cancel_common
from real.exp import two, two_positive, two_nonzero, exists_nat_gt, inverse_pos, one_half_lt_one, div_two_is_mul_half
from real.continuity_base import continuous, continuous_at, uniform, uniform_condition, add_real_eps_between
from real.integral import integral, is_integrable, interval_contains, interval_set, interval_image, interval_inf, interval_sup, interval_inf_spec, interval_sup_spec, lower_sum, upper_sum, lower_sum_set, upper_sum_set, lower_sum_contains, upper_sum_contains, partition_step_lower, partition_step_upper, is_partition, partition_start, partition_end, partition_mono, partition_point_in_interval, diff_step, telescope, partial_lte, image_lower_bound, interval_set_contains_left, interval_set_contains_right, interval_contains_left, interval_contains_right, interval_contains_mono, sub_nonneg, integral_spec, set_supremum_unique, set_infimum_unique, sup_le_of_upper_bound, trivial_partition, trivial_partition_is_partition, trivial_partition_zero, trivial_partition_one, neg_lte_flip, set_infimum_is_lower_bound, set_lower_bound_contains_le, set_lower_bound_le_infimum, has_lower_bound, is_bounded_on, scalar_diff_step, scalar_diff_step_eq_mul_fn, const_lower_sum, constant_integrable, integral_const
from real.integral import neg_upper_bound_of_lower, negate_set_nonempty, inf_of_neg_sup, interval_inf_const, interval_sup_const
from real.supremum import completeness, is_nonempty, has_upper_bound, is_set_supremum, is_set_upper_bound, is_set_infimum, is_set_lower_bound, set_member_le_supremum, set_supremum_le_upper_bound, set_supremum_is_upper_bound, set_upper_bound_contains_le, function_image, function_image_contains, negate_set, negate_set_contains
from real.integral_exp import uniform_partition, uniform_partition_start, uniform_partition_end, uniform_partition_monotone, uniform_partition_is_partition, uniform_partition_width, image_upper_bound, div_nonneg_pos_denom, from_nat_lte_mono, mul_frac_right, nonneg_frac_all_n_imp_zero, add_sub_cancel_shift, sub_add_cancel_shift, add_one_mul_sub
from real.real_base import gt_zero_imp_pos, pos_gt_zero, lte_lt_trans, close_imp_bounds, lt_add_right, add_comm, add_assoc, neg_neg, neg_distrib
from real.real_ring import mul_pos_pos, lt_mul_pos_left, from_nat_is_from_rat, mul_sub_distrib_left
from real.derivative_continuity import div_mul_cancel_denominator
from real.derivative_trig import abs_of_nonneg
from real.am_gm import div_pos_of_pos_pos, partial_const
from real.harmonic import from_nat_suc_pos_real
from real.double_sum import sub_seq, partial_sub_seq
from ordered_field import mul_le_mul_of_nonneg_right, mul_lt_mul_of_pos_right
from algebra.add_ordered_group import add_le_add, add_le_add_right
from algebra.semigroup import mul_fn
from data.basic.set import Set, maps_into_set_image
from data.basic.functions import compose
from list import partial, partial_one, partial_pointwise_eq, partial_scalar_mul, partial_split_last, partial_drop_first

numerals Real
numerals Nat

// ---------------------------------------------------------------------------
// Small order lemmas
// ---------------------------------------------------------------------------

/// A real that is strictly below every positive real is at most zero.
theorem lt_all_positive_eps_imp_lte_zero(x: Real) {
    (forall(eps: Real) { eps.is_positive implies x < eps })
    implies x <= Real.0
} by {
    if forall(eps: Real) { eps.is_positive implies x < eps } {
        if not x <= Real.0 {
            not_lte_imp_gt[Real](x, Real.0)
            Real.0 < x
            gt_zero_imp_pos(x)
            x.is_positive
            forall(eps0: Real) {
                eps0.is_positive implies x < eps0
            }
            x.is_positive implies x < x
            x < x
            not_lt_self(x)
            false
        }
        x <= Real.0
    }
}

// ---------------------------------------------------------------------------
// Bounded functions: the same-partition inequality
// ---------------------------------------------------------------------------

/// The infimum of a bounded function on [x, y] is at most its supremum.
theorem interval_inf_le_interval_sup(f: Real -> Real, x: Real, y: Real, lb: Real, ub: Real) {
    x <= y and (forall(t: Real) { interval_contains(x, y, t) implies lb <= f(t) }) and (forall(t: Real) { interval_contains(x, y, t) implies f(t) <= ub })
    implies interval_inf(f, x, y) <= interval_sup(f, x, y)
} by {
    if x <= y and (forall(t: Real) { interval_contains(x, y, t) implies lb <= f(t) }) and (forall(t: Real) { interval_contains(x, y, t) implies f(t) <= ub }) {
        interval_set_contains_left(x, y)
        is_nonempty(interval_set(x, y))
        image_lower_bound(f, x, y, lb)
        is_set_lower_bound(interval_image(f, x, y), lb)
        exists(bb: Real) {
            is_set_lower_bound(interval_image(f, x, y), bb)
        }
        has_lower_bound(interval_image(f, x, y))
        is_nonempty(interval_set(x, y)) and has_lower_bound(interval_image(f, x, y))
        interval_inf_spec(f, x, y)
        is_set_infimum(interval_image(f, x, y), interval_inf(f, x, y))
        set_infimum_is_lower_bound(interval_image(f, x, y), interval_inf(f, x, y))
        is_set_lower_bound(interval_image(f, x, y), interval_inf(f, x, y))
        maps_into_set_image(interval_set(x, y), f, x)
        function_image(f, interval_set(x, y)).contains(f(x))
        interval_image(f, x, y).contains(f(x))
        set_lower_bound_contains_le(interval_image(f, x, y), interval_inf(f, x, y), f(x))
        interval_inf(f, x, y) <= f(x)
        image_upper_bound(f, x, y, ub)
        is_set_upper_bound(interval_image(f, x, y), ub)
        exists(bb0: Real) {
            is_set_upper_bound(interval_image(f, x, y), bb0)
        }
        has_upper_bound(interval_image(f, x, y))
        is_nonempty(interval_set(x, y)) and has_upper_bound(interval_image(f, x, y))
        interval_sup_spec(f, x, y)
        is_set_supremum(interval_image(f, x, y), interval_sup(f, x, y))
        set_upper_bound_contains_le(interval_image(f, x, y), interval_sup(f, x, y), f(x))
        f(x) <= interval_sup(f, x, y)
        lte_trans[Real](interval_inf(f, x, y), f(x), interval_sup(f, x, y))
        interval_inf(f, x, y) <= interval_sup(f, x, y)
    }
}

/// The lower Darboux step of a bounded function over a subinterval of a
/// partition of [a, b] is at most its upper Darboux step there.
theorem partition_step_lower_le_upper(f: Real -> Real, p: Nat -> Real, a: Real, b: Real, n: Nat, k: Nat, lb: Real, ub: Real) {
    a <= b and is_partition(p, a, b, n) and k < n and (forall(t: Real) { interval_contains(a, b, t) implies lb <= f(t) }) and (forall(t: Real) { interval_contains(a, b, t) implies f(t) <= ub })
    implies partition_step_lower(f, p, k) <= partition_step_upper(f, p, k)
} by {
    if a <= b and is_partition(p, a, b, n) and k < n and (forall(t: Real) { interval_contains(a, b, t) implies lb <= f(t) }) and (forall(t: Real) { interval_contains(a, b, t) implies f(t) <= ub }) {
        lt_imp_lte_suc(k, n)
        k + 1 <= n
        k <= k + 1
        partition_mono(p, a, b, n, k, k + 1)
        p(k) <= p(k + 1)
        partition_point_in_interval(p, a, b, n, k)
        interval_contains(a, b, p(k))
        partition_point_in_interval(p, a, b, n, k + 1)
        interval_contains(a, b, p(k + 1))
        forall(t: Real) {
            if interval_contains(p(k), p(k + 1), t) {
                interval_contains_left(p(k), p(k + 1), t)
                p(k) <= t
                interval_contains_right(p(k), p(k + 1), t)
                t <= p(k + 1)
                interval_contains_mono(a, b, p(k), p(k + 1), t)
                interval_contains(a, b, t)
                forall(t0: Real) {
                    interval_contains(a, b, t0) implies lb <= f(t0)
                }
                interval_contains(a, b, t) implies lb <= f(t)
                lb <= f(t)
            }
        }
        forall(t: Real) {
            if interval_contains(p(k), p(k + 1), t) {
                interval_contains_left(p(k), p(k + 1), t)
                p(k) <= t
                interval_contains_right(p(k), p(k + 1), t)
                t <= p(k + 1)
                interval_contains_mono(a, b, p(k), p(k + 1), t)
                interval_contains(a, b, t)
                forall(t1: Real) {
                    interval_contains(a, b, t1) implies f(t1) <= ub
                }
                interval_contains(a, b, t) implies f(t) <= ub
                f(t) <= ub
            }
        }
        interval_set_contains_left(p(k), p(k + 1))
        is_nonempty(interval_set(p(k), p(k + 1)))
        image_lower_bound(f, p(k), p(k + 1), lb)
        is_set_lower_bound(interval_image(f, p(k), p(k + 1)), lb)
        exists(bb: Real) {
            is_set_lower_bound(interval_image(f, p(k), p(k + 1)), bb)
        }
        has_lower_bound(interval_image(f, p(k), p(k + 1)))
        is_nonempty(interval_set(p(k), p(k + 1))) and has_lower_bound(interval_image(f, p(k), p(k + 1)))
        interval_inf_spec(f, p(k), p(k + 1))
        is_set_infimum(interval_image(f, p(k), p(k + 1)), interval_inf(f, p(k), p(k + 1)))
        set_infimum_is_lower_bound(interval_image(f, p(k), p(k + 1)), interval_inf(f, p(k), p(k + 1)))
        is_set_lower_bound(interval_image(f, p(k), p(k + 1)), interval_inf(f, p(k), p(k + 1)))
        maps_into_set_image(interval_set(p(k), p(k + 1)), f, p(k))
        function_image(f, interval_set(p(k), p(k + 1))).contains(f(p(k)))
        interval_image(f, p(k), p(k + 1)).contains(f(p(k)))
        set_lower_bound_contains_le(interval_image(f, p(k), p(k + 1)), interval_inf(f, p(k), p(k + 1)), f(p(k)))
        interval_inf(f, p(k), p(k + 1)) <= f(p(k))
        image_upper_bound(f, p(k), p(k + 1), ub)
        is_set_upper_bound(interval_image(f, p(k), p(k + 1)), ub)
        exists(bb0: Real) {
            is_set_upper_bound(interval_image(f, p(k), p(k + 1)), bb0)
        }
        has_upper_bound(interval_image(f, p(k), p(k + 1)))
        is_nonempty(interval_set(p(k), p(k + 1))) and has_upper_bound(interval_image(f, p(k), p(k + 1)))
        interval_sup_spec(f, p(k), p(k + 1))
        is_set_supremum(interval_image(f, p(k), p(k + 1)), interval_sup(f, p(k), p(k + 1)))
        set_upper_bound_contains_le(interval_image(f, p(k), p(k + 1)), interval_sup(f, p(k), p(k + 1)), f(p(k)))
        f(p(k)) <= interval_sup(f, p(k), p(k + 1))
        lte_trans[Real](interval_inf(f, p(k), p(k + 1)), f(p(k)), interval_sup(f, p(k), p(k + 1)))
        interval_inf(f, p(k), p(k + 1)) <= interval_sup(f, p(k), p(k + 1))
        sub_nonneg(p(k), p(k + 1))
        Real.0 <= p(k + 1) - p(k)
        mul_le_mul_of_nonneg_right(interval_inf(f, p(k), p(k + 1)), interval_sup(f, p(k), p(k + 1)), p(k + 1) - p(k))
        interval_inf(f, p(k), p(k + 1)) * (p(k + 1) - p(k)) <= interval_sup(f, p(k), p(k + 1)) * (p(k + 1) - p(k))
        partition_step_lower(f, p, k) <= partition_step_upper(f, p, k)
    }
}

/// The lower Darboux sum of a bounded function over a partition of [a, b] is
/// at most its upper Darboux sum over the same partition.
theorem lower_sum_le_upper_sum(f: Real -> Real, p: Nat -> Real, a: Real, b: Real, n: Nat, lb: Real, ub: Real) {
    a <= b and is_partition(p, a, b, n) and (forall(t: Real) { interval_contains(a, b, t) implies lb <= f(t) }) and (forall(t: Real) { interval_contains(a, b, t) implies f(t) <= ub })
    implies lower_sum(f, p, n) <= upper_sum(f, p, n)
} by {
    if a <= b and is_partition(p, a, b, n) and (forall(t: Real) { interval_contains(a, b, t) implies lb <= f(t) }) and (forall(t: Real) { interval_contains(a, b, t) implies f(t) <= ub }) {
        forall(k: Nat) {
            if k < n {
                partition_step_lower_le_upper(f, p, a, b, n, k, lb, ub)
                partition_step_lower(f, p, k) <= partition_step_upper(f, p, k)
            }
        }
        partial_lte(partition_step_lower(f, p), partition_step_upper(f, p), n)
        partial(partition_step_lower(f, p), n) <= partial(partition_step_upper(f, p), n)
        lower_sum(f, p, n) <= upper_sum(f, p, n)
    }
}

// ---------------------------------------------------------------------------
// Bounded functions: the Darboux sum sets are bounded
// ---------------------------------------------------------------------------

/// The lower Darboux step of a bounded function over a subinterval of a
/// partition of [a, b] is at most ub times the width of that subinterval.
theorem partition_step_lower_le_ub_width(f: Real -> Real, p: Nat -> Real, a: Real, b: Real, n: Nat, k: Nat, lb: Real, ub: Real) {
    a <= b and is_partition(p, a, b, n) and k < n and (forall(t: Real) { interval_contains(a, b, t) implies lb <= f(t) }) and (forall(t: Real) { interval_contains(a, b, t) implies f(t) <= ub })
    implies partition_step_lower(f, p, k) <= ub * diff_step(p, k)
} by {
    if a <= b and is_partition(p, a, b, n) and k < n and (forall(t: Real) { interval_contains(a, b, t) implies lb <= f(t) }) and (forall(t: Real) { interval_contains(a, b, t) implies f(t) <= ub }) {
        lt_imp_lte_suc(k, n)
        k + 1 <= n
        k <= k + 1
        partition_mono(p, a, b, n, k, k + 1)
        p(k) <= p(k + 1)
        partition_point_in_interval(p, a, b, n, k)
        interval_contains(a, b, p(k))
        partition_point_in_interval(p, a, b, n, k + 1)
        interval_contains(a, b, p(k + 1))
        interval_set_contains_left(p(k), p(k + 1))
        is_nonempty(interval_set(p(k), p(k + 1)))
        forall(t: Real) {
            if interval_contains(p(k), p(k + 1), t) {
                interval_contains_left(p(k), p(k + 1), t)
                p(k) <= t
                interval_contains_right(p(k), p(k + 1), t)
                t <= p(k + 1)
                interval_contains_mono(a, b, p(k), p(k + 1), t)
                interval_contains(a, b, t)
                forall(t0: Real) {
                    interval_contains(a, b, t0) implies lb <= f(t0)
                }
                interval_contains(a, b, t) implies lb <= f(t)
                lb <= f(t)
            }
        }
        image_lower_bound(f, p(k), p(k + 1), lb)
        is_set_lower_bound(interval_image(f, p(k), p(k + 1)), lb)
        exists(bb: Real) {
            is_set_lower_bound(interval_image(f, p(k), p(k + 1)), bb)
        }
        has_lower_bound(interval_image(f, p(k), p(k + 1)))
        is_nonempty(interval_set(p(k), p(k + 1))) and has_lower_bound(interval_image(f, p(k), p(k + 1)))
        interval_inf_spec(f, p(k), p(k + 1))
        is_set_infimum(interval_image(f, p(k), p(k + 1)), interval_inf(f, p(k), p(k + 1)))
        set_infimum_is_lower_bound(interval_image(f, p(k), p(k + 1)), interval_inf(f, p(k), p(k + 1)))
        is_set_lower_bound(interval_image(f, p(k), p(k + 1)), interval_inf(f, p(k), p(k + 1)))
        maps_into_set_image(interval_set(p(k), p(k + 1)), f, p(k))
        function_image(f, interval_set(p(k), p(k + 1))).contains(f(p(k)))
        interval_image(f, p(k), p(k + 1)).contains(f(p(k)))
        set_lower_bound_contains_le(interval_image(f, p(k), p(k + 1)), interval_inf(f, p(k), p(k + 1)), f(p(k)))
        interval_inf(f, p(k), p(k + 1)) <= f(p(k))
        lt_imp_lte(k, n)
        k <= n
        partition_point_in_interval(p, a, b, n, k)
        interval_contains(a, b, p(k))
        forall(t1: Real) {
            interval_contains(a, b, t1) implies f(t1) <= ub
        }
        interval_contains(a, b, p(k)) implies f(p(k)) <= ub
        f(p(k)) <= ub
        lte_trans[Real](interval_inf(f, p(k), p(k + 1)), f(p(k)), ub)
        interval_inf(f, p(k), p(k + 1)) <= ub
        sub_nonneg(p(k), p(k + 1))
        Real.0 <= p(k + 1) - p(k)
        mul_le_mul_of_nonneg_right(interval_inf(f, p(k), p(k + 1)), ub, p(k + 1) - p(k))
        interval_inf(f, p(k), p(k + 1)) * (p(k + 1) - p(k)) <= ub * (p(k + 1) - p(k))
        partition_step_lower(f, p, k) <= ub * diff_step(p, k)
    }
}

/// Every lower Darboux sum of a bounded function is at most ub * (b - a).
theorem lower_sum_le_ub_width(f: Real -> Real, p: Nat -> Real, a: Real, b: Real, n: Nat, lb: Real, ub: Real) {
    a <= b and is_partition(p, a, b, n) and (forall(t: Real) { interval_contains(a, b, t) implies lb <= f(t) }) and (forall(t: Real) { interval_contains(a, b, t) implies f(t) <= ub })
    implies lower_sum(f, p, n) <= ub * (b - a)
} by {
    if a <= b and is_partition(p, a, b, n) and (forall(t: Real) { interval_contains(a, b, t) implies lb <= f(t) }) and (forall(t: Real) { interval_contains(a, b, t) implies f(t) <= ub }) {
        forall(k: Nat) {
            if k < n {
                partition_step_lower_le_ub_width(f, p, a, b, n, k, lb, ub)
                partition_step_lower(f, p, k) <= scalar_diff_step(ub, p, k)
            }
        }
        partial_lte(partition_step_lower(f, p), scalar_diff_step(ub, p), n)
        partial(partition_step_lower(f, p), n) <= partial(scalar_diff_step(ub, p), n)
        scalar_diff_step_eq_mul_fn(ub, p)
        scalar_diff_step(ub, p) = mul_fn(ub, diff_step(p))
        partial(scalar_diff_step(ub, p), n) = partial(mul_fn(ub, diff_step(p)), n)
        partial_scalar_mul(ub, diff_step(p), n)
        ub * partial(diff_step(p), n) = partial(mul_fn(ub, diff_step(p)), n)
        partial(scalar_diff_step(ub, p), n) = ub * partial(diff_step(p), n)
        telescope(p, n)
        partial(diff_step(p), n) = p(n) - p(Nat.0)
        partition_start(p, a, b, n)
        p(Nat.0) = a
        partition_end(p, a, b, n)
        p(n) = b
        p(n) - p(Nat.0) = b - a
        ub * (p(n) - p(Nat.0)) = ub * (b - a)
        partial(partition_step_lower(f, p), n) <= ub * (b - a)
        lower_sum(f, p, n) <= ub * (b - a)
    }
}

/// The upper Darboux step of a bounded function over a subinterval of a
/// partition of [a, b] is at least lb times the width of that subinterval.
theorem partition_step_upper_ge_lb_width(f: Real -> Real, p: Nat -> Real, a: Real, b: Real, n: Nat, k: Nat, lb: Real, ub: Real) {
    a <= b and is_partition(p, a, b, n) and k < n and (forall(t: Real) { interval_contains(a, b, t) implies lb <= f(t) }) and (forall(t: Real) { interval_contains(a, b, t) implies f(t) <= ub })
    implies lb * diff_step(p, k) <= partition_step_upper(f, p, k)
} by {
    if a <= b and is_partition(p, a, b, n) and k < n and (forall(t: Real) { interval_contains(a, b, t) implies lb <= f(t) }) and (forall(t: Real) { interval_contains(a, b, t) implies f(t) <= ub }) {
        lt_imp_lte_suc(k, n)
        k + 1 <= n
        k <= k + 1
        partition_mono(p, a, b, n, k, k + 1)
        p(k) <= p(k + 1)
        partition_point_in_interval(p, a, b, n, k)
        interval_contains(a, b, p(k))
        partition_point_in_interval(p, a, b, n, k + 1)
        interval_contains(a, b, p(k + 1))
        interval_set_contains_left(p(k), p(k + 1))
        is_nonempty(interval_set(p(k), p(k + 1)))
        forall(t: Real) {
            if interval_contains(p(k), p(k + 1), t) {
                interval_contains_left(p(k), p(k + 1), t)
                p(k) <= t
                interval_contains_right(p(k), p(k + 1), t)
                t <= p(k + 1)
                interval_contains_mono(a, b, p(k), p(k + 1), t)
                interval_contains(a, b, t)
                forall(t0: Real) {
                    interval_contains(a, b, t0) implies lb <= f(t0)
                }
                interval_contains(a, b, t) implies lb <= f(t)
                lb <= f(t)
            }
        }
        forall(t: Real) {
            if interval_contains(p(k), p(k + 1), t) {
                interval_contains_left(p(k), p(k + 1), t)
                p(k) <= t
                interval_contains_right(p(k), p(k + 1), t)
                t <= p(k + 1)
                interval_contains_mono(a, b, p(k), p(k + 1), t)
                interval_contains(a, b, t)
                forall(t1: Real) {
                    interval_contains(a, b, t1) implies f(t1) <= ub
                }
                interval_contains(a, b, t) implies f(t) <= ub
                f(t) <= ub
            }
        }
        image_upper_bound(f, p(k), p(k + 1), ub)
        is_set_upper_bound(interval_image(f, p(k), p(k + 1)), ub)
        exists(bb0: Real) {
            is_set_upper_bound(interval_image(f, p(k), p(k + 1)), bb0)
        }
        has_upper_bound(interval_image(f, p(k), p(k + 1)))
        is_nonempty(interval_set(p(k), p(k + 1))) and has_upper_bound(interval_image(f, p(k), p(k + 1)))
        interval_sup_spec(f, p(k), p(k + 1))
        is_set_supremum(interval_image(f, p(k), p(k + 1)), interval_sup(f, p(k), p(k + 1)))
        set_supremum_is_upper_bound(interval_image(f, p(k), p(k + 1)), interval_sup(f, p(k), p(k + 1)))
        is_set_upper_bound(interval_image(f, p(k), p(k + 1)), interval_sup(f, p(k), p(k + 1)))
        maps_into_set_image(interval_set(p(k), p(k + 1)), f, p(k))
        function_image(f, interval_set(p(k), p(k + 1))).contains(f(p(k)))
        interval_image(f, p(k), p(k + 1)).contains(f(p(k)))
        set_upper_bound_contains_le(interval_image(f, p(k), p(k + 1)), interval_sup(f, p(k), p(k + 1)), f(p(k)))
        f(p(k)) <= interval_sup(f, p(k), p(k + 1))
        lt_imp_lte(k, n)
        k <= n
        partition_point_in_interval(p, a, b, n, k)
        interval_contains(a, b, p(k))
        forall(t2: Real) {
            interval_contains(a, b, t2) implies lb <= f(t2)
        }
        interval_contains(a, b, p(k)) implies lb <= f(p(k))
        lb <= f(p(k))
        lte_trans[Real](lb, f(p(k)), interval_sup(f, p(k), p(k + 1)))
        lb <= interval_sup(f, p(k), p(k + 1))
        sub_nonneg(p(k), p(k + 1))
        Real.0 <= p(k + 1) - p(k)
        mul_le_mul_of_nonneg_right(lb, interval_sup(f, p(k), p(k + 1)), p(k + 1) - p(k))
        lb * (p(k + 1) - p(k)) <= interval_sup(f, p(k), p(k + 1)) * (p(k + 1) - p(k))
        lb * diff_step(p, k) <= partition_step_upper(f, p, k)
    }
}

/// Every upper Darboux sum of a bounded function is at least lb * (b - a).
theorem lb_width_le_upper_sum(f: Real -> Real, p: Nat -> Real, a: Real, b: Real, n: Nat, lb: Real, ub: Real) {
    a <= b and is_partition(p, a, b, n) and (forall(t: Real) { interval_contains(a, b, t) implies lb <= f(t) }) and (forall(t: Real) { interval_contains(a, b, t) implies f(t) <= ub })
    implies lb * (b - a) <= upper_sum(f, p, n)
} by {
    if a <= b and is_partition(p, a, b, n) and (forall(t: Real) { interval_contains(a, b, t) implies lb <= f(t) }) and (forall(t: Real) { interval_contains(a, b, t) implies f(t) <= ub }) {
        forall(k: Nat) {
            if k < n {
                partition_step_upper_ge_lb_width(f, p, a, b, n, k, lb, ub)
                scalar_diff_step(lb, p, k) <= partition_step_upper(f, p, k)
            }
        }
        partial_lte(scalar_diff_step(lb, p), partition_step_upper(f, p), n)
        partial(scalar_diff_step(lb, p), n) <= partial(partition_step_upper(f, p), n)
        scalar_diff_step_eq_mul_fn(lb, p)
        scalar_diff_step(lb, p) = mul_fn(lb, diff_step(p))
        partial(scalar_diff_step(lb, p), n) = partial(mul_fn(lb, diff_step(p)), n)
        partial_scalar_mul(lb, diff_step(p), n)
        lb * partial(diff_step(p), n) = partial(mul_fn(lb, diff_step(p)), n)
        partial(scalar_diff_step(lb, p), n) = lb * partial(diff_step(p), n)
        telescope(p, n)
        partial(diff_step(p), n) = p(n) - p(Nat.0)
        partition_start(p, a, b, n)
        p(Nat.0) = a
        partition_end(p, a, b, n)
        p(n) = b
        p(n) - p(Nat.0) = b - a
        lb * (p(n) - p(Nat.0)) = lb * (b - a)
        lb * (b - a) <= partial(partition_step_upper(f, p), n)
        lb * (b - a) <= upper_sum(f, p, n)
    }
}

// ---------------------------------------------------------------------------
// Bounded functions: the Darboux sum sets are bounded
// ---------------------------------------------------------------------------

/// The set of lower Darboux sums of a bounded function is bounded above.
theorem lower_sum_set_bounded_above(f: Real -> Real, a: Real, b: Real, lb: Real, ub: Real) {
    a <= b and (forall(t: Real) { interval_contains(a, b, t) implies lb <= f(t) }) and (forall(t: Real) { interval_contains(a, b, t) implies f(t) <= ub })
    implies is_set_upper_bound(lower_sum_set(f, a, b), ub * (b - a))
} by {
    if a <= b and (forall(t: Real) { interval_contains(a, b, t) implies lb <= f(t) }) and (forall(t: Real) { interval_contains(a, b, t) implies f(t) <= ub }) {
        forall(x: Real) {
            if lower_sum_set(f, a, b).contains(x) {
                lower_sum_set(f, a, b).contains(x) = lower_sum_contains(f, a, b, x)
                lower_sum_contains(f, a, b, x)
                let (p: Nat -> Real, n: Nat) satisfy {
                    is_partition(p, a, b, n) and x = lower_sum(f, p, n)
                }
                lower_sum_le_ub_width(f, p, a, b, n, lb, ub)
                lower_sum(f, p, n) <= ub * (b - a)
                x = lower_sum(f, p, n)
                x <= ub * (b - a)
            }
        }
        is_set_upper_bound(lower_sum_set(f, a, b), ub * (b - a))
    }
}

/// The set of upper Darboux sums of a bounded function is bounded below.
theorem upper_sum_set_bounded_below(f: Real -> Real, a: Real, b: Real, lb: Real, ub: Real) {
    a <= b and (forall(t: Real) { interval_contains(a, b, t) implies lb <= f(t) }) and (forall(t: Real) { interval_contains(a, b, t) implies f(t) <= ub })
    implies is_set_lower_bound(upper_sum_set(f, a, b), lb * (b - a))
} by {
    if a <= b and (forall(t: Real) { interval_contains(a, b, t) implies lb <= f(t) }) and (forall(t: Real) { interval_contains(a, b, t) implies f(t) <= ub }) {
        forall(x: Real) {
            if upper_sum_set(f, a, b).contains(x) {
                upper_sum_set(f, a, b).contains(x) = upper_sum_contains(f, a, b, x)
                upper_sum_contains(f, a, b, x)
                let (p: Nat -> Real, n: Nat) satisfy {
                    is_partition(p, a, b, n) and x = upper_sum(f, p, n)
                }
                lb_width_le_upper_sum(f, p, a, b, n, lb, ub)
                lb * (b - a) <= upper_sum(f, p, n)
                x = upper_sum(f, p, n)
                lb * (b - a) <= x
            }
        }
        is_set_lower_bound(upper_sum_set(f, a, b), lb * (b - a))
    }
}

/// The lower Darboux sums of a bounded function on [a, b] have a supremum.
theorem lower_sum_set_sup_exists_of_bounded(f: Real -> Real, a: Real, b: Real, lb: Real, ub: Real) {
    a <= b and (forall(t: Real) { interval_contains(a, b, t) implies lb <= f(t) }) and (forall(t: Real) { interval_contains(a, b, t) implies f(t) <= ub })
    implies exists(l: Real) {
        is_set_supremum(lower_sum_set(f, a, b), l)
    }
} by {
    if a <= b and (forall(t: Real) { interval_contains(a, b, t) implies lb <= f(t) }) and (forall(t: Real) { interval_contains(a, b, t) implies f(t) <= ub }) {
        trivial_partition_is_partition(a, b)
        is_partition(trivial_partition(a, b), a, b, Nat.1)
        is_partition(trivial_partition(a, b), a, b, Nat.1) and lower_sum(f, trivial_partition(a, b), Nat.1) = lower_sum(f, trivial_partition(a, b), Nat.1)
        exists(p: Nat -> Real, n: Nat) {
            is_partition(p, a, b, n) and lower_sum(f, trivial_partition(a, b), Nat.1) = lower_sum(f, p, n)
        }
        lower_sum_contains(f, a, b, lower_sum(f, trivial_partition(a, b), Nat.1))
        lower_sum_set(f, a, b).contains(lower_sum(f, trivial_partition(a, b), Nat.1)) = lower_sum_contains(f, a, b, lower_sum(f, trivial_partition(a, b), Nat.1))
        lower_sum_set(f, a, b).contains(lower_sum(f, trivial_partition(a, b), Nat.1))
        exists(x: Real) {
            lower_sum_set(f, a, b).contains(x)
        }
        is_nonempty(lower_sum_set(f, a, b))
        lower_sum_set_bounded_above(f, a, b, lb, ub)
        is_set_upper_bound(lower_sum_set(f, a, b), ub * (b - a))
        exists(bb0: Real) {
            is_set_upper_bound(lower_sum_set(f, a, b), bb0)
        }
        has_upper_bound(lower_sum_set(f, a, b))
        is_nonempty(lower_sum_set(f, a, b)) and has_upper_bound(lower_sum_set(f, a, b))
        completeness(lower_sum_set(f, a, b))
        let l: Real satisfy {
            is_set_supremum(lower_sum_set(f, a, b), l)
        }
        exists(l2: Real) {
            is_set_supremum(lower_sum_set(f, a, b), l2)
        }
    }
}

/// The upper Darboux sums of a bounded function on [a, b] have an infimum.
theorem upper_sum_set_inf_exists_of_bounded(f: Real -> Real, a: Real, b: Real, lb: Real, ub: Real) {
    a <= b and (forall(t: Real) { interval_contains(a, b, t) implies lb <= f(t) }) and (forall(t: Real) { interval_contains(a, b, t) implies f(t) <= ub })
    implies exists(u: Real) {
        is_set_infimum(upper_sum_set(f, a, b), u)
    }
} by {
    if a <= b and (forall(t: Real) { interval_contains(a, b, t) implies lb <= f(t) }) and (forall(t: Real) { interval_contains(a, b, t) implies f(t) <= ub }) {
        trivial_partition_is_partition(a, b)
        is_partition(trivial_partition(a, b), a, b, Nat.1)
        is_partition(trivial_partition(a, b), a, b, Nat.1) and upper_sum(f, trivial_partition(a, b), Nat.1) = upper_sum(f, trivial_partition(a, b), Nat.1)
        exists(p: Nat -> Real, n: Nat) {
            is_partition(p, a, b, n) and upper_sum(f, trivial_partition(a, b), Nat.1) = upper_sum(f, p, n)
        }
        upper_sum_contains(f, a, b, upper_sum(f, trivial_partition(a, b), Nat.1))
        upper_sum_set(f, a, b).contains(upper_sum(f, trivial_partition(a, b), Nat.1)) = upper_sum_contains(f, a, b, upper_sum(f, trivial_partition(a, b), Nat.1))
        upper_sum_set(f, a, b).contains(upper_sum(f, trivial_partition(a, b), Nat.1))
        exists(x: Real) {
            upper_sum_set(f, a, b).contains(x)
        }
        is_nonempty(upper_sum_set(f, a, b))
        negate_set_nonempty(upper_sum_set(f, a, b))
        is_nonempty(negate_set(upper_sum_set(f, a, b)))
        upper_sum_set_bounded_below(f, a, b, lb, ub)
        is_set_lower_bound(upper_sum_set(f, a, b), lb * (b - a))
        neg_upper_bound_of_lower(upper_sum_set(f, a, b), lb * (b - a))
        is_set_upper_bound(negate_set(upper_sum_set(f, a, b)), -(lb * (b - a)))
        exists(bb0: Real) {
            is_set_upper_bound(negate_set(upper_sum_set(f, a, b)), bb0)
        }
        has_upper_bound(negate_set(upper_sum_set(f, a, b)))
        is_nonempty(negate_set(upper_sum_set(f, a, b))) and has_upper_bound(negate_set(upper_sum_set(f, a, b)))
        completeness(negate_set(upper_sum_set(f, a, b)))
        let sup: Real satisfy {
            is_set_supremum(negate_set(upper_sum_set(f, a, b)), sup)
        }
        inf_of_neg_sup(upper_sum_set(f, a, b), sup)
        is_set_infimum(upper_sum_set(f, a, b), -sup)
        exists(u2: Real) {
            is_set_infimum(upper_sum_set(f, a, b), u2)
        }
    }
}

// ---------------------------------------------------------------------------
// The Darboux criterion
// ---------------------------------------------------------------------------

/// True if some partition of [a, b] makes the difference of the upper and
/// lower Darboux sums of f less than eps.
define darboux_partition_exists(f: Real -> Real, a: Real, b: Real, eps: Real) -> Bool {
    exists(p: Nat -> Real, n: Nat) {
        is_partition(p, a, b, n) and upper_sum(f, p, n) - lower_sum(f, p, n) < eps
    }
}

/// True if the upper and lower Darboux sums of f over [a, b] can be made
/// arbitrarily close over a single partition.
define darboux_condition(f: Real -> Real, a: Real, b: Real) -> Bool {
    forall(eps: Real) {
        eps.is_positive implies darboux_partition_exists(f, a, b, eps)
    }
}

/// The eps-condition forces the infimum of the upper sums below the supremum
/// of the lower sums.
theorem darboux_eps_imp_inf_le_sup(f: Real -> Real, a: Real, b: Real, l: Real, u: Real) {
    is_set_supremum(lower_sum_set(f, a, b), l) and is_set_infimum(upper_sum_set(f, a, b), u) and darboux_condition(f, a, b)
    implies u <= l
} by {
    if is_set_supremum(lower_sum_set(f, a, b), l) and is_set_infimum(upper_sum_set(f, a, b), u) and darboux_condition(f, a, b) {
        forall(eps: Real) {
            if eps.is_positive {
                darboux_condition(f, a, b) = forall(e: Real) {
                    e.is_positive implies darboux_partition_exists(f, a, b, e)
                }
                forall(e: Real) {
                    e.is_positive implies darboux_partition_exists(f, a, b, e)
                }
                eps.is_positive implies darboux_partition_exists(f, a, b, eps)
                darboux_partition_exists(f, a, b, eps)
                darboux_partition_exists(f, a, b, eps) = exists(p: Nat -> Real, n: Nat) {
                    is_partition(p, a, b, n) and upper_sum(f, p, n) - lower_sum(f, p, n) < eps
                }
                exists(p: Nat -> Real, n: Nat) {
                    is_partition(p, a, b, n) and upper_sum(f, p, n) - lower_sum(f, p, n) < eps
                }
                let (p: Nat -> Real, n: Nat) satisfy {
                    is_partition(p, a, b, n) and upper_sum(f, p, n) - lower_sum(f, p, n) < eps
                }
                is_partition(p, a, b, n) and lower_sum(f, p, n) = lower_sum(f, p, n)
                exists(p0: Nat -> Real, n0: Nat) {
                    is_partition(p0, a, b, n0) and lower_sum(f, p, n) = lower_sum(f, p0, n0)
                }
                lower_sum_contains(f, a, b, lower_sum(f, p, n))
                lower_sum_set(f, a, b).contains(lower_sum(f, p, n)) = lower_sum_contains(f, a, b, lower_sum(f, p, n))
                lower_sum_set(f, a, b).contains(lower_sum(f, p, n))
                set_member_le_supremum(lower_sum_set(f, a, b), l, lower_sum(f, p, n))
                lower_sum(f, p, n) <= l
                is_partition(p, a, b, n) and upper_sum(f, p, n) = upper_sum(f, p, n)
                exists(p1: Nat -> Real, n1: Nat) {
                    is_partition(p1, a, b, n1) and upper_sum(f, p, n) = upper_sum(f, p1, n1)
                }
                upper_sum_contains(f, a, b, upper_sum(f, p, n))
                upper_sum_set(f, a, b).contains(upper_sum(f, p, n)) = upper_sum_contains(f, a, b, upper_sum(f, p, n))
                upper_sum_set(f, a, b).contains(upper_sum(f, p, n))
                set_infimum_is_lower_bound(upper_sum_set(f, a, b), u)
                is_set_lower_bound(upper_sum_set(f, a, b), u)
                set_lower_bound_contains_le(upper_sum_set(f, a, b), u, upper_sum(f, p, n))
                u <= upper_sum(f, p, n)
                neg_lte_flip(lower_sum(f, p, n), l)
                -l <= -lower_sum(f, p, n)
                add_le_add(u, upper_sum(f, p, n), -l, -lower_sum(f, p, n))
                u + -l <= upper_sum(f, p, n) + -lower_sum(f, p, n)
                u - l = u + -l
                upper_sum(f, p, n) - lower_sum(f, p, n) = upper_sum(f, p, n) + -lower_sum(f, p, n)
                u - l <= upper_sum(f, p, n) - lower_sum(f, p, n)
                upper_sum(f, p, n) - lower_sum(f, p, n) < eps
                lt_of_lte_of_lt(u - l, upper_sum(f, p, n) - lower_sum(f, p, n), eps)
                u - l < eps
            }
        }
        lt_all_positive_eps_imp_lte_zero(u - l)
        u - l <= Real.0
        add_le_add_right[Real](u - l, Real.0, l)
        (u - l) + l <= Real.0 + l
        (u - l) + l = u
        Real.0 + l = l
        u <= l
    }
}

/// The Darboux criterion, in conditional form: if f is bounded on [a, b], its
/// upper and lower Darboux sums can be made arbitrarily close over a single
/// partition, and every lower sum is at most every upper sum (the
/// cross-partition comparison, an explicit hypothesis), then f is integrable
/// on [a, b].
theorem darboux_criterion(f: Real -> Real, a: Real, b: Real, lb: Real, ub: Real) {
    a <= b and (forall(t: Real) { interval_contains(a, b, t) implies lb <= f(t) }) and (forall(t: Real) { interval_contains(a, b, t) implies f(t) <= ub }) and darboux_condition(f, a, b) and (forall(l: Real, u: Real) {
        is_set_supremum(lower_sum_set(f, a, b), l) and is_set_infimum(upper_sum_set(f, a, b), u)
        implies l <= u
    })
    implies is_integrable(f, a, b)
} by {
    if a <= b and (forall(t: Real) { interval_contains(a, b, t) implies lb <= f(t) }) and (forall(t: Real) { interval_contains(a, b, t) implies f(t) <= ub }) and darboux_condition(f, a, b) and (forall(l: Real, u: Real) {
           is_set_supremum(lower_sum_set(f, a, b), l) and is_set_infimum(upper_sum_set(f, a, b), u)
           implies l <= u
       }) {
        lower_sum_set_sup_exists_of_bounded(f, a, b, lb, ub)
        let l: Real satisfy {
            is_set_supremum(lower_sum_set(f, a, b), l)
        }
        upper_sum_set_inf_exists_of_bounded(f, a, b, lb, ub)
        let u: Real satisfy {
            is_set_infimum(upper_sum_set(f, a, b), u)
        }
        darboux_eps_imp_inf_le_sup(f, a, b, l, u)
        u <= l
        forall(l0: Real, u0: Real) {
            is_set_supremum(lower_sum_set(f, a, b), l0) and is_set_infimum(upper_sum_set(f, a, b), u0) implies l0 <= u0
        }
        is_set_supremum(lower_sum_set(f, a, b), l) and is_set_infimum(upper_sum_set(f, a, b), u) implies l <= u
        l <= u
        lte_antisymm[Real](l, u)
        l = u
        is_set_supremum(lower_sum_set(f, a, b), l) and is_set_infimum(upper_sum_set(f, a, b), l)
        exists(m: Real) {
            is_set_supremum(lower_sum_set(f, a, b), m) and is_set_infimum(upper_sum_set(f, a, b), m)
        }
        is_integrable(f, a, b)
    }
}

// ---------------------------------------------------------------------------
// Uniform continuity gives the Darboux eps-condition
// ---------------------------------------------------------------------------
//
// A uniformly continuous function is the classical source of the Darboux
// eps-condition: on the cells of a fine enough uniform partition the
// oscillation of f is bounded by the uniform tolerance, so the upper and
// lower Darboux sums differ by at most the tolerance times the total width.

/// A strict inequality a < b makes b - a positive.
theorem lt_imp_pos_sub(a: Real, b: Real) {
    a < b implies Real.0 < b - a
} by {
    if a < b {
        lt_add_right(a, b, -a)
        a + -a < b + -a
        a + -a = Real.0
        b + -a = b - a
        Real.0 < b - a
    }
}

/// For every positive d and every c, some fraction c / (n + 1) is below d.
theorem exists_n_frac_lt(c: Real, d: Real) {
    d.is_positive implies exists(n: Nat) {
        c / from_nat[Real](n.suc) < d
    }
} by {
    if d.is_positive {
        if c <= Real.0 {
            from_nat[Real](Nat.0.suc) = from_nat[Real](Nat.1)
            from_nat[Real](Nat.1) = Real.1
            from_nat[Real](Nat.0.suc) = Real.1
            c / from_nat[Real](Nat.0.suc) = c / Real.1
            c / Real.1 = c
            c / from_nat[Real](Nat.0.suc) = c
            c <= Real.0
            pos_gt_zero(d)
            Real.0 < d
            lt_of_lte_of_lt(c, Real.0, d)
            c < d
            c / from_nat[Real](Nat.0.suc) < d
            exists(nn: Nat) {
                c / from_nat[Real](nn.suc) < d
            }
        } else {
            not c <= Real.0
            not_lte_imp_gt[Real](c, Real.0)
            Real.0 < c
            gt_zero_imp_pos(c)
            c.is_positive
            pos_gt_zero(d)
            d > Real.0
            inverse_pos(d)
            d.inverse.is_positive
            mul_pos_pos(c, d.inverse)
            (c * d.inverse).is_positive
            pos_gt_zero(c * d.inverse)
            c * d.inverse > Real.0
            c / d = c * d.inverse
            c / d > Real.0
            gt_zero_imp_pos(c / d)
            (c / d).is_positive
            exists_nat_gt(c / d)
            let n: Nat satisfy {
                Real.from_rat(Rat.from_nat(n)) > c / d
            }
            from_nat_is_from_rat(n)
            from_nat[Real](n) = Real.from_rat(Rat.from_nat(n))
            from_nat[Real](n) > c / d
            c / d < from_nat[Real](n)
            n <= n.suc
            from_nat_lte_mono(n, n.suc)
            from_nat[Real](n) <= from_nat[Real](n.suc)
            lt_of_lt_of_lte(c / d, from_nat[Real](n), from_nat[Real](n.suc))
            c / d < from_nat[Real](n.suc)
            lt_mul_pos_left(c / d, from_nat[Real](n.suc), d)
            d * (c / d) < d * from_nat[Real](n.suc)
            d != Real.0
            div_mul_cancel_denominator(c, d)
            ((c / d) * d) = c
            d * (c / d) = c
            c < d * from_nat[Real](n.suc)
            from_nat_suc_pos_real(n)
            from_nat[Real](n.suc) > Real.0
            inverse_pos(from_nat[Real](n.suc))
            from_nat[Real](n.suc).inverse.is_positive
            lt_mul_pos_left(c, d * from_nat[Real](n.suc), from_nat[Real](n.suc).inverse)
            from_nat[Real](n.suc).inverse * c < from_nat[Real](n.suc).inverse * (d * from_nat[Real](n.suc))
            from_nat[Real](n.suc).inverse * c = c * from_nat[Real](n.suc).inverse
            add_comm(d, from_nat[Real](n.suc))
            d * from_nat[Real](n.suc) = from_nat[Real](n.suc) * d
            from_nat[Real](n.suc).inverse * (d * from_nat[Real](n.suc)) = from_nat[Real](n.suc).inverse * (from_nat[Real](n.suc) * d)
            from_nat[Real](n.suc).inverse * (from_nat[Real](n.suc) * d) = (from_nat[Real](n.suc).inverse * from_nat[Real](n.suc)) * d
            mul_inverse(from_nat[Real](n.suc))
            from_nat[Real](n.suc) * from_nat[Real](n.suc).inverse = Real.1
            from_nat[Real](n.suc).inverse * from_nat[Real](n.suc) = Real.1
            (from_nat[Real](n.suc).inverse * from_nat[Real](n.suc)) * d = Real.1 * d
            Real.1 * d = d
            from_nat[Real](n.suc).inverse * (d * from_nat[Real](n.suc)) = d
            c * from_nat[Real](n.suc).inverse < d
            c / from_nat[Real](n.suc) = c * from_nat[Real](n.suc).inverse
            c / from_nat[Real](n.suc) < d
            exists(nn: Nat) {
                c / from_nat[Real](nn.suc) < d
            }
        }
    }
}

/// On a cell of the uniform partition whose width is below the uniform
/// continuity modulus, the oscillation of f (supremum minus infimum of the
/// cell) is at most twice the tolerance.
/// Multiplying by two is adding a real to itself.
theorem two_mul(e: Real) {
    two * e = e + e
} by {
    two = Real.1 + Real.1
    two * e = (Real.1 + Real.1) * e
    (Real.1 + Real.1) * e = e + e
}

/// (x + e) - (x - e) is two times e.
theorem add_eps_sub_eps(x: Real, e: Real) {
    (x + e) - (x - e) = two * e
} by {
    two_mul(e)
    two * e = e + e
    x - e = x + -e
    (x + e) - (x - e) = (x + e) + -(x + -e)
    neg_distrib(x, -e)
    -(x + -e) = -x + -(-e)
    neg_neg(e)
    -(-e) = e
    -(x + -e) = -x + e
    (x + e) + -(x + -e) = (x + e) + (-x + e)
    add_assoc(x, e, -x)
    (x + e) + -x = x + (e + -x)
    add_comm(e, -x)
    e + -x = -x + e
    x + (e + -x) = x + (-x + e)
    (x + e) + -x = x + (-x + e)
    add_assoc(x, -x, e)
    (x + -x) + e = x + (-x + e)
    x + -x = Real.0
    (x + -x) + e = Real.0 + e
    Real.0 + e = e
    (x + e) + -x = e
    add_assoc(x + e, -x, e)
    ((x + e) + -x) + e = (x + e) + (-x + e)
    ((x + e) + -x) + e = e + e
    (x + e) + (-x + e) = e + e
    (x + e) - (x - e) = e + e
    (x + e) - (x - e) = two * e
}

theorem uniform_cell_oscillation(f: Real -> Real, a: Real, b: Real, n: Nat, i: Nat, delta: Real, eps1: Real) {
    a <= b and i < n.suc and delta.is_positive and eps1.is_positive and uniform_condition(f, delta, eps1) and (b - a) / from_nat[Real](n.suc) < delta
    implies interval_sup(f, uniform_partition(a, b, n, i), uniform_partition(a, b, n, i + 1)) - interval_inf(f, uniform_partition(a, b, n, i), uniform_partition(a, b, n, i + 1)) <= two * eps1
} by {
    if a <= b and i < n.suc and delta.is_positive and eps1.is_positive and uniform_condition(f, delta, eps1) and (b - a) / from_nat[Real](n.suc) < delta {
        lt_imp_lte_suc(i, n.suc)
        i + 1 <= n.suc
        i <= i + 1
        uniform_partition_monotone(a, b, n, i, i + 1)
        uniform_partition(a, b, n, i) <= uniform_partition(a, b, n, i + 1)
        uniform_partition_width(a, b, n, i)
        uniform_partition(a, b, n, i + 1) - uniform_partition(a, b, n, i) = (b - a) / from_nat[Real](n.suc)
        forall(t: Real) {
            if interval_contains(uniform_partition(a, b, n, i), uniform_partition(a, b, n, i + 1), t) {
                interval_contains_left(uniform_partition(a, b, n, i), uniform_partition(a, b, n, i + 1), t)
                uniform_partition(a, b, n, i) <= t
                interval_contains_right(uniform_partition(a, b, n, i), uniform_partition(a, b, n, i + 1), t)
                t <= uniform_partition(a, b, n, i + 1)
                sub_nonneg(uniform_partition(a, b, n, i), t)
                Real.0 <= t - uniform_partition(a, b, n, i)
                abs_of_nonneg(t - uniform_partition(a, b, n, i))
                (t - uniform_partition(a, b, n, i)).abs = t - uniform_partition(a, b, n, i)
                add_le_add_right[Real](t, uniform_partition(a, b, n, i + 1), -uniform_partition(a, b, n, i))
                t + -uniform_partition(a, b, n, i) <= uniform_partition(a, b, n, i + 1) + -uniform_partition(a, b, n, i)
                t - uniform_partition(a, b, n, i) <= uniform_partition(a, b, n, i + 1) - uniform_partition(a, b, n, i)
                uniform_partition(a, b, n, i + 1) - uniform_partition(a, b, n, i) = (b - a) / from_nat[Real](n.suc)
                t - uniform_partition(a, b, n, i) <= (b - a) / from_nat[Real](n.suc)
                (b - a) / from_nat[Real](n.suc) < delta
                lt_of_lte_of_lt(t - uniform_partition(a, b, n, i), (b - a) / from_nat[Real](n.suc), delta)
                t - uniform_partition(a, b, n, i) < delta
                (t - uniform_partition(a, b, n, i)).abs < delta
                t.is_close(uniform_partition(a, b, n, i), delta)
                uniform_condition(f, delta, eps1) = forall(r1: Real, r2: Real) {
                    r1.is_close(r2, delta) implies f(r1).is_close(f(r2), eps1)
                }
                forall(r1: Real, r2: Real) {
                    r1.is_close(r2, delta) implies f(r1).is_close(f(r2), eps1)
                }
                t.is_close(uniform_partition(a, b, n, i), delta) implies f(t).is_close(f(uniform_partition(a, b, n, i)), eps1)
                f(t).is_close(f(uniform_partition(a, b, n, i)), eps1)
                close_imp_bounds(f(t), f(uniform_partition(a, b, n, i)), eps1)
                f(uniform_partition(a, b, n, i)) - eps1 < f(t)
                f(t) < f(uniform_partition(a, b, n, i)) + eps1
                lt_imp_lte(f(uniform_partition(a, b, n, i)) - eps1, f(t))
                f(uniform_partition(a, b, n, i)) - eps1 <= f(t)
                lt_imp_lte(f(t), f(uniform_partition(a, b, n, i)) + eps1)
                f(t) <= f(uniform_partition(a, b, n, i)) + eps1
                f(uniform_partition(a, b, n, i)) - eps1 <= f(t) and f(t) <= f(uniform_partition(a, b, n, i)) + eps1
            }
        }
        forall(t: Real) {
            if interval_contains(uniform_partition(a, b, n, i), uniform_partition(a, b, n, i + 1), t) {
                interval_contains_left(uniform_partition(a, b, n, i), uniform_partition(a, b, n, i + 1), t)
                uniform_partition(a, b, n, i) <= t
                interval_contains_right(uniform_partition(a, b, n, i), uniform_partition(a, b, n, i + 1), t)
                t <= uniform_partition(a, b, n, i + 1)
                sub_nonneg(uniform_partition(a, b, n, i), t)
                Real.0 <= t - uniform_partition(a, b, n, i)
                abs_of_nonneg(t - uniform_partition(a, b, n, i))
                (t - uniform_partition(a, b, n, i)).abs = t - uniform_partition(a, b, n, i)
                add_le_add_right[Real](t, uniform_partition(a, b, n, i + 1), -uniform_partition(a, b, n, i))
                t + -uniform_partition(a, b, n, i) <= uniform_partition(a, b, n, i + 1) + -uniform_partition(a, b, n, i)
                t - uniform_partition(a, b, n, i) <= uniform_partition(a, b, n, i + 1) - uniform_partition(a, b, n, i)
                uniform_partition(a, b, n, i + 1) - uniform_partition(a, b, n, i) = (b - a) / from_nat[Real](n.suc)
                t - uniform_partition(a, b, n, i) <= (b - a) / from_nat[Real](n.suc)
                (b - a) / from_nat[Real](n.suc) < delta
                lt_of_lte_of_lt(t - uniform_partition(a, b, n, i), (b - a) / from_nat[Real](n.suc), delta)
                t - uniform_partition(a, b, n, i) < delta
                (t - uniform_partition(a, b, n, i)).abs < delta
                t.is_close(uniform_partition(a, b, n, i), delta)
                uniform_condition(f, delta, eps1) = forall(r1: Real, r2: Real) {
                    r1.is_close(r2, delta) implies f(r1).is_close(f(r2), eps1)
                }
                forall(r1: Real, r2: Real) {
                    r1.is_close(r2, delta) implies f(r1).is_close(f(r2), eps1)
                }
                t.is_close(uniform_partition(a, b, n, i), delta) implies f(t).is_close(f(uniform_partition(a, b, n, i)), eps1)
                f(t).is_close(f(uniform_partition(a, b, n, i)), eps1)
                close_imp_bounds(f(t), f(uniform_partition(a, b, n, i)), eps1)
                f(uniform_partition(a, b, n, i)) - eps1 < f(t)
                f(t) < f(uniform_partition(a, b, n, i)) + eps1
                lt_imp_lte(f(t), f(uniform_partition(a, b, n, i)) + eps1)
                f(t) <= f(uniform_partition(a, b, n, i)) + eps1
            }
        }
        forall(t: Real) {
            if interval_contains(uniform_partition(a, b, n, i), uniform_partition(a, b, n, i + 1), t) {
                interval_contains_left(uniform_partition(a, b, n, i), uniform_partition(a, b, n, i + 1), t)
                uniform_partition(a, b, n, i) <= t
                interval_contains_right(uniform_partition(a, b, n, i), uniform_partition(a, b, n, i + 1), t)
                t <= uniform_partition(a, b, n, i + 1)
                sub_nonneg(uniform_partition(a, b, n, i), t)
                Real.0 <= t - uniform_partition(a, b, n, i)
                abs_of_nonneg(t - uniform_partition(a, b, n, i))
                (t - uniform_partition(a, b, n, i)).abs = t - uniform_partition(a, b, n, i)
                add_le_add_right[Real](t, uniform_partition(a, b, n, i + 1), -uniform_partition(a, b, n, i))
                t + -uniform_partition(a, b, n, i) <= uniform_partition(a, b, n, i + 1) + -uniform_partition(a, b, n, i)
                t - uniform_partition(a, b, n, i) <= uniform_partition(a, b, n, i + 1) - uniform_partition(a, b, n, i)
                uniform_partition(a, b, n, i + 1) - uniform_partition(a, b, n, i) = (b - a) / from_nat[Real](n.suc)
                t - uniform_partition(a, b, n, i) <= (b - a) / from_nat[Real](n.suc)
                (b - a) / from_nat[Real](n.suc) < delta
                lt_of_lte_of_lt(t - uniform_partition(a, b, n, i), (b - a) / from_nat[Real](n.suc), delta)
                t - uniform_partition(a, b, n, i) < delta
                (t - uniform_partition(a, b, n, i)).abs < delta
                t.is_close(uniform_partition(a, b, n, i), delta)
                uniform_condition(f, delta, eps1) = forall(r1: Real, r2: Real) {
                    r1.is_close(r2, delta) implies f(r1).is_close(f(r2), eps1)
                }
                forall(r1: Real, r2: Real) {
                    r1.is_close(r2, delta) implies f(r1).is_close(f(r2), eps1)
                }
                t.is_close(uniform_partition(a, b, n, i), delta) implies f(t).is_close(f(uniform_partition(a, b, n, i)), eps1)
                f(t).is_close(f(uniform_partition(a, b, n, i)), eps1)
                close_imp_bounds(f(t), f(uniform_partition(a, b, n, i)), eps1)
                f(uniform_partition(a, b, n, i)) - eps1 < f(t)
                f(t) < f(uniform_partition(a, b, n, i)) + eps1
                lt_imp_lte(f(uniform_partition(a, b, n, i)) - eps1, f(t))
                f(uniform_partition(a, b, n, i)) - eps1 <= f(t)
            }
        }
        interval_set_contains_left(uniform_partition(a, b, n, i), uniform_partition(a, b, n, i + 1))
        is_nonempty(interval_set(uniform_partition(a, b, n, i), uniform_partition(a, b, n, i + 1)))
        image_lower_bound(f, uniform_partition(a, b, n, i), uniform_partition(a, b, n, i + 1), f(uniform_partition(a, b, n, i)) - eps1)
        is_set_lower_bound(interval_image(f, uniform_partition(a, b, n, i), uniform_partition(a, b, n, i + 1)), f(uniform_partition(a, b, n, i)) - eps1)
        exists(bb: Real) {
            is_set_lower_bound(interval_image(f, uniform_partition(a, b, n, i), uniform_partition(a, b, n, i + 1)), bb)
        }
        has_lower_bound(interval_image(f, uniform_partition(a, b, n, i), uniform_partition(a, b, n, i + 1)))
        is_nonempty(interval_set(uniform_partition(a, b, n, i), uniform_partition(a, b, n, i + 1))) and has_lower_bound(interval_image(f, uniform_partition(a, b, n, i), uniform_partition(a, b, n, i + 1)))
        interval_inf_spec(f, uniform_partition(a, b, n, i), uniform_partition(a, b, n, i + 1))
        is_set_infimum(interval_image(f, uniform_partition(a, b, n, i), uniform_partition(a, b, n, i + 1)), interval_inf(f, uniform_partition(a, b, n, i), uniform_partition(a, b, n, i + 1)))
        image_upper_bound(f, uniform_partition(a, b, n, i), uniform_partition(a, b, n, i + 1), f(uniform_partition(a, b, n, i)) + eps1)
        is_set_upper_bound(interval_image(f, uniform_partition(a, b, n, i), uniform_partition(a, b, n, i + 1)), f(uniform_partition(a, b, n, i)) + eps1)
        exists(bb0: Real) {
            is_set_upper_bound(interval_image(f, uniform_partition(a, b, n, i), uniform_partition(a, b, n, i + 1)), bb0)
        }
        has_upper_bound(interval_image(f, uniform_partition(a, b, n, i), uniform_partition(a, b, n, i + 1)))
        is_nonempty(interval_set(uniform_partition(a, b, n, i), uniform_partition(a, b, n, i + 1))) and has_upper_bound(interval_image(f, uniform_partition(a, b, n, i), uniform_partition(a, b, n, i + 1)))
        interval_sup_spec(f, uniform_partition(a, b, n, i), uniform_partition(a, b, n, i + 1))
        is_set_supremum(interval_image(f, uniform_partition(a, b, n, i), uniform_partition(a, b, n, i + 1)), interval_sup(f, uniform_partition(a, b, n, i), uniform_partition(a, b, n, i + 1)))
        set_lower_bound_le_infimum(interval_image(f, uniform_partition(a, b, n, i), uniform_partition(a, b, n, i + 1)), interval_inf(f, uniform_partition(a, b, n, i), uniform_partition(a, b, n, i + 1)), f(uniform_partition(a, b, n, i)) - eps1)
        f(uniform_partition(a, b, n, i)) - eps1 <= interval_inf(f, uniform_partition(a, b, n, i), uniform_partition(a, b, n, i + 1))
        set_supremum_le_upper_bound(interval_image(f, uniform_partition(a, b, n, i), uniform_partition(a, b, n, i + 1)), interval_sup(f, uniform_partition(a, b, n, i), uniform_partition(a, b, n, i + 1)), f(uniform_partition(a, b, n, i)) + eps1)
        interval_sup(f, uniform_partition(a, b, n, i), uniform_partition(a, b, n, i + 1)) <= f(uniform_partition(a, b, n, i)) + eps1
        neg_lte_flip(f(uniform_partition(a, b, n, i)) - eps1, interval_inf(f, uniform_partition(a, b, n, i), uniform_partition(a, b, n, i + 1)))
        -interval_inf(f, uniform_partition(a, b, n, i), uniform_partition(a, b, n, i + 1)) <= -(f(uniform_partition(a, b, n, i)) - eps1)
        add_le_add(interval_sup(f, uniform_partition(a, b, n, i), uniform_partition(a, b, n, i + 1)), f(uniform_partition(a, b, n, i)) + eps1, -interval_inf(f, uniform_partition(a, b, n, i), uniform_partition(a, b, n, i + 1)), -(f(uniform_partition(a, b, n, i)) - eps1))
        interval_sup(f, uniform_partition(a, b, n, i), uniform_partition(a, b, n, i + 1)) + -interval_inf(f, uniform_partition(a, b, n, i), uniform_partition(a, b, n, i + 1)) <= (f(uniform_partition(a, b, n, i)) + eps1) + -(f(uniform_partition(a, b, n, i)) - eps1)
        interval_sup(f, uniform_partition(a, b, n, i), uniform_partition(a, b, n, i + 1)) - interval_inf(f, uniform_partition(a, b, n, i), uniform_partition(a, b, n, i + 1)) = interval_sup(f, uniform_partition(a, b, n, i), uniform_partition(a, b, n, i + 1)) + -interval_inf(f, uniform_partition(a, b, n, i), uniform_partition(a, b, n, i + 1))
        interval_sup(f, uniform_partition(a, b, n, i), uniform_partition(a, b, n, i + 1)) + -interval_inf(f, uniform_partition(a, b, n, i), uniform_partition(a, b, n, i + 1)) <= (f(uniform_partition(a, b, n, i)) + eps1) + -(f(uniform_partition(a, b, n, i)) - eps1)
        interval_sup(f, uniform_partition(a, b, n, i), uniform_partition(a, b, n, i + 1)) - interval_inf(f, uniform_partition(a, b, n, i), uniform_partition(a, b, n, i + 1)) <= (f(uniform_partition(a, b, n, i)) + eps1) + -(f(uniform_partition(a, b, n, i)) - eps1)
        (f(uniform_partition(a, b, n, i)) + eps1) + -(f(uniform_partition(a, b, n, i)) - eps1) = (f(uniform_partition(a, b, n, i)) + eps1) - (f(uniform_partition(a, b, n, i)) - eps1)
        add_eps_sub_eps(f(uniform_partition(a, b, n, i)), eps1)
        (f(uniform_partition(a, b, n, i)) + eps1) - (f(uniform_partition(a, b, n, i)) - eps1) = two * eps1
        interval_sup(f, uniform_partition(a, b, n, i), uniform_partition(a, b, n, i + 1)) - interval_inf(f, uniform_partition(a, b, n, i), uniform_partition(a, b, n, i + 1)) <= two * eps1
    }
}

/// The difference of the upper and lower Darboux sums of f over the uniform
/// partition of [a, b] is at most twice the tolerance times the width, when
/// the mesh is below the uniform continuity modulus.
theorem uniform_upper_minus_lower_osc(f: Real -> Real, a: Real, b: Real, n: Nat, delta: Real, eps1: Real) {
    a <= b and delta.is_positive and eps1.is_positive and uniform_condition(f, delta, eps1) and (b - a) / from_nat[Real](n.suc) < delta
    implies upper_sum(f, uniform_partition(a, b, n), n.suc) - lower_sum(f, uniform_partition(a, b, n), n.suc) <= two * eps1 * (b - a)
} by {
    if a <= b and delta.is_positive and eps1.is_positive and uniform_condition(f, delta, eps1) and (b - a) / from_nat[Real](n.suc) < delta {
        uniform_partition_is_partition(a, b, n)
        is_partition(uniform_partition(a, b, n), a, b, n.suc)
        forall(i: Nat) {
            if i < n.suc {
                uniform_cell_oscillation(f, a, b, n, i, delta, eps1)
                interval_sup(f, uniform_partition(a, b, n, i), uniform_partition(a, b, n, i + 1)) - interval_inf(f, uniform_partition(a, b, n, i), uniform_partition(a, b, n, i + 1)) <= two * eps1
                lt_imp_lte_suc(i, n.suc)
                i + 1 <= n.suc
                i <= i + 1
                uniform_partition_monotone(a, b, n, i, i + 1)
                uniform_partition(a, b, n, i) <= uniform_partition(a, b, n, i + 1)
                sub_nonneg(uniform_partition(a, b, n, i), uniform_partition(a, b, n, i + 1))
                Real.0 <= uniform_partition(a, b, n, i + 1) - uniform_partition(a, b, n, i)
                mul_le_mul_of_nonneg_right(interval_sup(f, uniform_partition(a, b, n, i), uniform_partition(a, b, n, i + 1)) - interval_inf(f, uniform_partition(a, b, n, i), uniform_partition(a, b, n, i + 1)), two * eps1, uniform_partition(a, b, n, i + 1) - uniform_partition(a, b, n, i))
                (interval_sup(f, uniform_partition(a, b, n, i), uniform_partition(a, b, n, i + 1)) - interval_inf(f, uniform_partition(a, b, n, i), uniform_partition(a, b, n, i + 1))) * (uniform_partition(a, b, n, i + 1) - uniform_partition(a, b, n, i)) <= two * eps1 * (uniform_partition(a, b, n, i + 1) - uniform_partition(a, b, n, i))
                partition_step_upper(f, uniform_partition(a, b, n), i) - partition_step_lower(f, uniform_partition(a, b, n), i) = (interval_sup(f, uniform_partition(a, b, n, i), uniform_partition(a, b, n, i + 1)) - interval_inf(f, uniform_partition(a, b, n, i), uniform_partition(a, b, n, i + 1))) * (uniform_partition(a, b, n, i + 1) - uniform_partition(a, b, n, i))
                partition_step_upper(f, uniform_partition(a, b, n), i) - partition_step_lower(f, uniform_partition(a, b, n), i) <= two * eps1 * (uniform_partition(a, b, n, i + 1) - uniform_partition(a, b, n, i))
                uniform_partition_width(a, b, n, i)
                uniform_partition(a, b, n, i + 1) - uniform_partition(a, b, n, i) = (b - a) / from_nat[Real](n.suc)
                two * eps1 * (uniform_partition(a, b, n, i + 1) - uniform_partition(a, b, n, i)) = two * eps1 * ((b - a) / from_nat[Real](n.suc))
                partition_step_upper(f, uniform_partition(a, b, n), i) - partition_step_lower(f, uniform_partition(a, b, n), i) <= two * eps1 * ((b - a) / from_nat[Real](n.suc))
                scalar_diff_step(two * eps1, uniform_partition(a, b, n), i) = two * eps1 * ((b - a) / from_nat[Real](n.suc))
                partition_step_upper(f, uniform_partition(a, b, n), i) - partition_step_lower(f, uniform_partition(a, b, n), i) <= scalar_diff_step(two * eps1, uniform_partition(a, b, n), i)
            }
        }
        partial_lte(sub_seq(partition_step_upper(f, uniform_partition(a, b, n)), partition_step_lower(f, uniform_partition(a, b, n))), scalar_diff_step(two * eps1, uniform_partition(a, b, n)), n.suc)
        partial(sub_seq(partition_step_upper(f, uniform_partition(a, b, n)), partition_step_lower(f, uniform_partition(a, b, n))), n.suc) <= partial(scalar_diff_step(two * eps1, uniform_partition(a, b, n)), n.suc)
        partial_sub_seq(partition_step_upper(f, uniform_partition(a, b, n)), partition_step_lower(f, uniform_partition(a, b, n)), n.suc)
        partial(sub_seq(partition_step_upper(f, uniform_partition(a, b, n)), partition_step_lower(f, uniform_partition(a, b, n))), n.suc) = partial(partition_step_upper(f, uniform_partition(a, b, n)), n.suc) - partial(partition_step_lower(f, uniform_partition(a, b, n)), n.suc)
        partial(partition_step_upper(f, uniform_partition(a, b, n)), n.suc) - partial(partition_step_lower(f, uniform_partition(a, b, n)), n.suc) <= partial(scalar_diff_step(two * eps1, uniform_partition(a, b, n)), n.suc)
        upper_sum(f, uniform_partition(a, b, n), n.suc) = partial(partition_step_upper(f, uniform_partition(a, b, n)), n.suc)
        lower_sum(f, uniform_partition(a, b, n), n.suc) = partial(partition_step_lower(f, uniform_partition(a, b, n)), n.suc)
        upper_sum(f, uniform_partition(a, b, n), n.suc) - lower_sum(f, uniform_partition(a, b, n), n.suc) <= partial(scalar_diff_step(two * eps1, uniform_partition(a, b, n)), n.suc)
        scalar_diff_step_eq_mul_fn(two * eps1, uniform_partition(a, b, n))
        scalar_diff_step(two * eps1, uniform_partition(a, b, n)) = mul_fn(two * eps1, diff_step(uniform_partition(a, b, n)))
        partial(scalar_diff_step(two * eps1, uniform_partition(a, b, n)), n.suc) = partial(mul_fn(two * eps1, diff_step(uniform_partition(a, b, n))), n.suc)
        partial_scalar_mul(two * eps1, diff_step(uniform_partition(a, b, n)), n.suc)
        two * eps1 * partial(diff_step(uniform_partition(a, b, n)), n.suc) = partial(mul_fn(two * eps1, diff_step(uniform_partition(a, b, n))), n.suc)
        partial(scalar_diff_step(two * eps1, uniform_partition(a, b, n)), n.suc) = two * eps1 * partial(diff_step(uniform_partition(a, b, n)), n.suc)
        telescope(uniform_partition(a, b, n), n.suc)
        partial(diff_step(uniform_partition(a, b, n)), n.suc) = uniform_partition(a, b, n, n.suc) - uniform_partition(a, b, n, Nat.0)
        uniform_partition_end(a, b, n)
        uniform_partition(a, b, n, n.suc) = b
        uniform_partition_start(a, b, n)
        uniform_partition(a, b, n, Nat.0) = a
        uniform_partition(a, b, n, n.suc) - uniform_partition(a, b, n, Nat.0) = b - a
        two * eps1 * (uniform_partition(a, b, n, n.suc) - uniform_partition(a, b, n, Nat.0)) = two * eps1 * (b - a)
        partial(scalar_diff_step(two * eps1, uniform_partition(a, b, n)), n.suc) = two * eps1 * (b - a)
        upper_sum(f, uniform_partition(a, b, n), n.suc) - lower_sum(f, uniform_partition(a, b, n), n.suc) <= two * eps1 * (b - a)
    }
}

/// A uniformly continuous, bounded function on [a, b] satisfies the Darboux
/// eps-condition: its upper and lower Darboux sums can be made arbitrarily
/// close over a single partition.
theorem uniform_bounded_imp_darboux_condition(f: Real -> Real, a: Real, b: Real, lb: Real, ub: Real) {
    a <= b and uniform(f) and
    (forall(t: Real) { interval_contains(a, b, t) implies lb <= f(t) }) and
    (forall(t: Real) { interval_contains(a, b, t) implies f(t) <= ub })
    implies
    darboux_condition(f, a, b)
} by {
    if a <= b and uniform(f) and
       (forall(t: Real) { interval_contains(a, b, t) implies lb <= f(t) }) and
       (forall(t: Real) { interval_contains(a, b, t) implies f(t) <= ub }) {
        forall(eps: Real) {
            if eps.is_positive {
                if b = a {
                    trivial_partition_is_partition(a, b)
                    is_partition(trivial_partition(a, b), a, b, Nat.1)
                    partial_one(partition_step_upper(f, trivial_partition(a, b)))
                    partial(partition_step_upper(f, trivial_partition(a, b)), Nat.1) = partition_step_upper(f, trivial_partition(a, b), Nat.0)
                    partition_step_upper(f, trivial_partition(a, b), Nat.0) = interval_sup(f, trivial_partition(a, b, Nat.0), trivial_partition(a, b, Nat.1)) * diff_step(trivial_partition(a, b), Nat.0)
                    trivial_partition_zero(a, b)
                    trivial_partition(a, b, Nat.0) = a
                    trivial_partition_one(a, b)
                    trivial_partition(a, b, Nat.1) = b
                    interval_sup(f, trivial_partition(a, b, Nat.0), trivial_partition(a, b, Nat.1)) = interval_sup(f, a, b)
                    partition_step_upper(f, trivial_partition(a, b), Nat.0) = interval_sup(f, a, b) * diff_step(trivial_partition(a, b), Nat.0)
                    diff_step(trivial_partition(a, b), Nat.0) = trivial_partition(a, b, Nat.1) - trivial_partition(a, b, Nat.0)
                    trivial_partition(a, b, Nat.1) - trivial_partition(a, b, Nat.0) = b - a
                    b - a = a - a
                    a - a = Real.0
                    diff_step(trivial_partition(a, b), Nat.0) = Real.0
                    interval_sup(f, a, b) * diff_step(trivial_partition(a, b), Nat.0) = interval_sup(f, a, b) * Real.0
                    interval_sup(f, a, b) * Real.0 = Real.0
                    partition_step_upper(f, trivial_partition(a, b), Nat.0) = Real.0
                    partial(partition_step_upper(f, trivial_partition(a, b)), Nat.1) = Real.0
                    upper_sum(f, trivial_partition(a, b), Nat.1) = partial(partition_step_upper(f, trivial_partition(a, b)), Nat.1)
                    upper_sum(f, trivial_partition(a, b), Nat.1) = Real.0
                    partial_one(partition_step_lower(f, trivial_partition(a, b)))
                    partial(partition_step_lower(f, trivial_partition(a, b)), Nat.1) = partition_step_lower(f, trivial_partition(a, b), Nat.0)
                    partition_step_lower(f, trivial_partition(a, b), Nat.0) = interval_inf(f, trivial_partition(a, b, Nat.0), trivial_partition(a, b, Nat.1)) * diff_step(trivial_partition(a, b), Nat.0)
                    interval_inf(f, trivial_partition(a, b, Nat.0), trivial_partition(a, b, Nat.1)) = interval_inf(f, a, b)
                    partition_step_lower(f, trivial_partition(a, b), Nat.0) = interval_inf(f, a, b) * diff_step(trivial_partition(a, b), Nat.0)
                    interval_inf(f, a, b) * diff_step(trivial_partition(a, b), Nat.0) = interval_inf(f, a, b) * Real.0
                    interval_inf(f, a, b) * Real.0 = Real.0
                    partition_step_lower(f, trivial_partition(a, b), Nat.0) = Real.0
                    partial(partition_step_lower(f, trivial_partition(a, b)), Nat.1) = Real.0
                    lower_sum(f, trivial_partition(a, b), Nat.1) = partial(partition_step_lower(f, trivial_partition(a, b)), Nat.1)
                    lower_sum(f, trivial_partition(a, b), Nat.1) = Real.0
                    upper_sum(f, trivial_partition(a, b), Nat.1) - lower_sum(f, trivial_partition(a, b), Nat.1) = Real.0 - Real.0
                    Real.0 - Real.0 = Real.0
                    upper_sum(f, trivial_partition(a, b), Nat.1) - lower_sum(f, trivial_partition(a, b), Nat.1) = Real.0
                    pos_gt_zero(eps)
                    Real.0 < eps
                    upper_sum(f, trivial_partition(a, b), Nat.1) - lower_sum(f, trivial_partition(a, b), Nat.1) < eps
                    is_partition(trivial_partition(a, b), a, b, Nat.1) and upper_sum(f, trivial_partition(a, b), Nat.1) - lower_sum(f, trivial_partition(a, b), Nat.1) < eps
                    exists(p: Nat -> Real, n0: Nat) {
                        is_partition(p, a, b, n0) and upper_sum(f, p, n0) - lower_sum(f, p, n0) < eps
                    }
                } else {
                    not b = a
                    a != b
                    lt_of_lte_of_ne(a, b)
                    a < b
                    let eps1 = eps / (two * (two * (b - a)))
                    lt_imp_pos_sub(a, b)
                    Real.0 < b - a
                    gt_zero_imp_pos(b - a)
                    (b - a).is_positive
                    two_positive
                    two.is_positive
                    mul_pos_pos(two, b - a)
                    (two * (b - a)).is_positive
                    mul_pos_pos(two, two * (b - a))
                    (two * (two * (b - a))).is_positive
                    pos_gt_zero(eps)
                    eps > Real.0
                    pos_gt_zero(two * (two * (b - a)))
                    two * (two * (b - a)) > Real.0
                    div_pos_of_pos_pos(eps, two * (two * (b - a)))
                    eps / (two * (two * (b - a))) > Real.0
                    eps1 > Real.0
                    gt_zero_imp_pos(eps1)
                    eps1.is_positive
                    uniform(f) = forall(e: Real) {
                        e.is_positive implies exists(d: Real) {
                            d.is_positive and uniform_condition(f, d, e)
                        }
                    }
                    forall(e: Real) {
                        e.is_positive implies exists(d: Real) {
                            d.is_positive and uniform_condition(f, d, e)
                        }
                    }
                    eps1.is_positive implies exists(d: Real) {
                        d.is_positive and uniform_condition(f, d, eps1)
                    }
                    exists(d: Real) {
                        d.is_positive and uniform_condition(f, d, eps1)
                    }
                    let delta: Real satisfy {
                        delta.is_positive and uniform_condition(f, delta, eps1)
                    }
                    delta.is_positive
                    exists_n_frac_lt(b - a, delta)
                    let n: Nat satisfy {
                        (b - a) / from_nat[Real](n.suc) < delta
                    }
                    (b - a) / from_nat[Real](n.suc) < delta
                    uniform_upper_minus_lower_osc(f, a, b, n, delta, eps1)
                    upper_sum(f, uniform_partition(a, b, n), n.suc) - lower_sum(f, uniform_partition(a, b, n), n.suc) <= two * eps1 * (b - a)
                    eps1 = eps / (two * (two * (b - a)))
                    two * eps1 * (b - a) = (two * (b - a)) * eps1
                    (two * (b - a)) * eps1 = (two * (b - a)) * (eps / (two * (two * (b - a))))
                    mul_frac_right(eps, two * (two * (b - a)), two * (b - a))
                    (eps / (two * (two * (b - a)))) * (two * (b - a)) = (eps * (two * (b - a))) / (two * (two * (b - a)))
                    (two * (b - a)) * (eps / (two * (two * (b - a)))) = (eps * (two * (b - a))) / (two * (two * (b - a)))
                    two * eps1 * (b - a) = (eps * (two * (b - a))) / (two * (two * (b - a)))
                    two_nonzero
                    two != Real.0
                    two * (b - a) != Real.0
                    div_cancel_common(eps, two * (b - a), two)
                    (eps * (two * (b - a))) / (two * (two * (b - a))) = eps / two
                    two * eps1 * (b - a) = eps / two
                    div_two_is_mul_half(eps)
                    eps / two = eps * Real.one_half
                    two * eps1 * (b - a) = eps * Real.one_half
                    one_half_lt_one
                    Real.one_half < Real.1
                    mul_lt_mul_of_pos_right(Real.one_half, Real.1, eps)
                    Real.one_half * eps < Real.1 * eps
                    Real.one_half * eps = eps * Real.one_half
                    Real.1 * eps = eps
                    eps * Real.one_half < eps
                    two * eps1 * (b - a) < eps
                    lt_of_lte_of_lt(upper_sum(f, uniform_partition(a, b, n), n.suc) - lower_sum(f, uniform_partition(a, b, n), n.suc), two * eps1 * (b - a), eps)
                    upper_sum(f, uniform_partition(a, b, n), n.suc) - lower_sum(f, uniform_partition(a, b, n), n.suc) < eps
                    uniform_partition_is_partition(a, b, n)
                    is_partition(uniform_partition(a, b, n), a, b, n.suc)
                    is_partition(uniform_partition(a, b, n), a, b, n.suc) and upper_sum(f, uniform_partition(a, b, n), n.suc) - lower_sum(f, uniform_partition(a, b, n), n.suc) < eps
                    exists(p: Nat -> Real, n0: Nat) {
                        is_partition(p, a, b, n0) and upper_sum(f, p, n0) - lower_sum(f, p, n0) < eps
                    }
                }
            }
        }
        forall(eps: Real) {
            eps.is_positive implies darboux_partition_exists(f, a, b, eps)
        }
        darboux_condition(f, a, b)
    }
}

/// A uniformly continuous, bounded function on [a, b] is integrable, provided
/// every lower Darboux sum is at most every upper Darboux sum (the
/// cross-partition comparison, an explicit hypothesis).
theorem uniform_bounded_integrable(f: Real -> Real, a: Real, b: Real, lb: Real, ub: Real) {
    a <= b and uniform(f) and
    (forall(t: Real) { interval_contains(a, b, t) implies lb <= f(t) }) and
    (forall(t: Real) { interval_contains(a, b, t) implies f(t) <= ub }) and
    (forall(l: Real, u: Real) {
        is_set_supremum(lower_sum_set(f, a, b), l) and is_set_infimum(upper_sum_set(f, a, b), u)
        implies l <= u
    })
    implies
    is_integrable(f, a, b)
} by {
    if a <= b and uniform(f) and
       (forall(t: Real) { interval_contains(a, b, t) implies lb <= f(t) }) and
       (forall(t: Real) { interval_contains(a, b, t) implies f(t) <= ub }) and
       (forall(l: Real, u: Real) {
           is_set_supremum(lower_sum_set(f, a, b), l) and is_set_infimum(upper_sum_set(f, a, b), u)
           implies l <= u
       }) {
        uniform_bounded_imp_darboux_condition(f, a, b, lb, ub)
        darboux_condition(f, a, b)
        darboux_criterion(f, a, b, lb, ub)
        is_integrable(f, a, b)
    }
}

// ---------------------------------------------------------------------------
// The classical criteria, stated
// ---------------------------------------------------------------------------
//
// The remaining classical integrability criteria are stated below.  Each
// needs an ingredient that the library does not yet provide:
//   - continuous_imp_integrable needs the extreme value theorem (continuity
//     on [a, b] implies boundedness), Heine-Cantor (continuity on [a, b]
//     implies uniform continuity on [a, b]), and the cross-partition
//     comparison of Darboux sums,
//   - finite_discontinuities_integrable needs the same cross-partition
//     comparison, together with a finitary account of the discontinuity set,
//   - riemann_lebesgue needs the Lebesgue measure machinery of lebesgue.ac
//     (whose finite-additivity layer is still under construction).
// The theorems above (the Darboux criterion in conditional form and the
// uniform-continuity route to the eps-condition) are exactly the shared core
// of these classical proofs, minus the three missing bridges.

/// The set of points at which f is discontinuous.
define discontinuity_set(f: Real -> Real) -> Set[Real] {
    Set[Real].new(function(x: Real) {
        not continuous_at(f, x)
    })
}

/// True if the set s has Lebesgue measure zero, i.e. its Lebesgue outer
/// measure vanishes.
///
/// Note: this definition is stated as a comment because the Lebesgue measure
/// of lebesgue.ac is built over the `Real` type of the `real` interface
/// module, which does not unify with the `Real` of real.real_field used by
/// the Darboux integral here; reconciling the two parallel developments of
/// the reals is future work.
//
// define has_lebesgue_measure_zero(s: Set[Real]) -> Bool {
//     lebesgue_outer_measure(s) = Real.0
// }

// The Riemann-Lebesgue criterion: a bounded function on [a, b] is integrable
// exactly when its discontinuity set restricted to [a, b] has Lebesgue
// measure zero.  Stated here for the record; the proof needs the Lebesgue
// measure machinery of lebesgue.ac (countable subadditivity and the
// Caratheodory criterion), which is still under construction there.
//
// theorem riemann_lebesgue(f: Real -> Real, a: Real, b: Real) {
//     a <= b and is_bounded_on(f, a, b)
//     implies
//     (is_integrable(f, a, b) =
//         has_lebesgue_measure_zero(discontinuity_set(f).intersection(interval_set(a, b))))
// }

// A bounded function with only finitely many discontinuities on [a, b] is
// integrable.  The classical proof surrounds each discontinuity with a small
// interval (f is bounded on the remainder, where it is continuous, and
// bounded functions are integrable on the remainder by the Darboux
// criterion), then shrinks the surrounding intervals; it needs the
// cross-partition comparison of Darboux sums (the explicit hypothesis in
// darboux_criterion above) and a finitary account of the discontinuity set.
//
// theorem finite_discontinuities_integrable(f: Real -> Real, a: Real, b: Real) {
//     a <= b and is_bounded_on(f, a, b) and has_finitely_many_discontinuities(f, a, b)
//     implies is_integrable(f, a, b)
// }

// A continuous function on [a, b] is integrable.  The three missing bridges
// are: the extreme value theorem (continuity on [a, b] implies boundedness),
// Heine-Cantor (continuity on [a, b] implies uniform continuity on [a, b];
// the library only has global uniform continuity), and the cross-partition
// comparison of Darboux sums (any lower sum is at most any upper sum).  With
// those in place, uniform_bounded_imp_darboux_condition and darboux_criterion
// above complete the proof.
//
// theorem continuous_imp_integrable(f: Real -> Real, a: Real, b: Real) {
//     a <= b and continuous(f) implies is_integrable(f, a, b)
// }

// The cross-partition comparison of Darboux sums, the missing hypothesis of
// darboux_criterion.  Its proof needs the common-refinement machinery of
// partitions (refining a partition increases lower sums and decreases upper
// sums), which is not yet in the library; the single-partition case
// lower_sum_le_upper_sum above is proved.
//
// theorem cross_partition_lower_le_upper(f: Real -> Real, p: Nat -> Real, q: Nat -> Real, a: Real, b: Real, n: Nat, m: Nat, lb: Real, ub: Real) {
//     a <= b and is_partition(p, a, b, n) and is_partition(q, a, b, m) and
//     (forall(t: Real) { interval_contains(a, b, t) implies lb <= f(t) }) and
//     (forall(t: Real) { interval_contains(a, b, t) implies f(t) <= ub })
//     implies
//     lower_sum(f, p, n) <= upper_sum(f, q, m)
// }

// Additivity of the integral over adjacent intervals:
// integral(f, a, c) = integral(f, a, b) + integral(f, b, c) for a <= b <= c.
// The proof compares Darboux sums on [a, c] with the concatenation of
// partitions of [a, b] and [b, c]; it needs the cross-partition comparison
// of Darboux sums and the restriction of integrability to subintervals.
//
// theorem integral_additivity(f: Real -> Real, a: Real, b: Real, c: Real) {
//     a <= b and b <= c and is_integrable(f, a, c)
//     implies
//     integral(f, a, c) = integral(f, a, b) + integral(f, b, c)
// }

// ---------------------------------------------------------------------------
// Functions constant on the open interval
// ---------------------------------------------------------------------------
//
// A bounded function that is constant on the open interval (a, b) (its value
// at the endpoints is arbitrary) is integrable, with integral c * (b - a).
// Every lower Darboux sum is at most c * (b - a) and every upper sum at
// least it, because every nondegenerate subinterval of a partition contains
// an interior point of (a, b); the uniform partitions then force the Darboux
// eps-condition, since only the first and last cells can carry oscillation.

/// A nondegenerate subinterval of [a, b] contains a point strictly inside
/// (a, b), where f takes the value c.
theorem interior_constant_cell_point(f: Real -> Real, a: Real, b: Real, c: Real, x: Real, y: Real) {
    a <= x and x < y and y <= b and
    (forall(t: Real) { a < t and t < b implies f(t) = c })
    implies
    exists(t: Real) {
        interval_contains(x, y, t) and f(t) = c
    }
} by {
    if a <= x and x < y and y <= b and (forall(t: Real) { a < t and t < b implies f(t) = c }) {
        if x = a {
            if y = b {
                a < b
                add_real_eps_between(a, b)
                exists(eps: Real) {
                    eps.is_positive and a + eps < b
                }
                let eps: Real satisfy {
                    eps.is_positive and a + eps < b
                }
                lt_add_right(Real.0, eps, a)
                Real.0 + a < eps + a
                Real.0 + a = a
                eps + a = a + eps
                a < a + eps
                lt_imp_lte(a, a + eps)
                a <= a + eps
                lt_imp_lte(a + eps, b)
                a + eps <= b
                interval_contains(a, b, a + eps)
                interval_contains(x, y, a + eps)
                a < a + eps and a + eps < b
                forall(t0: Real) {
                    a < t0 and t0 < b implies f(t0) = c
                }
                a < a + eps and a + eps < b implies f(a + eps) = c
                f(a + eps) = c
                interval_contains(x, y, a + eps) and f(a + eps) = c
                exists(t: Real) {
                    interval_contains(x, y, t) and f(t) = c
                }
            } else {
                not y = b
                lt_of_lte_of_ne(y, b)
                y < b
                interval_contains(x, y, y)
                a < y
                a < y and y < b
                forall(t1: Real) {
                    a < t1 and t1 < b implies f(t1) = c
                }
                a < y and y < b implies f(y) = c
                f(y) = c
                interval_contains(x, y, y) and f(y) = c
                exists(t: Real) {
                    interval_contains(x, y, t) and f(t) = c
                }
            }
        } else {
            not x = a
            lt_of_lte_of_ne(a, x)
            a < x
            interval_contains(x, y, x)
            x < y and y <= b
            lt_of_lt_of_lte(x, y, b)
            x < b
            a < x and x < b
            forall(t2: Real) {
                a < t2 and t2 < b implies f(t2) = c
            }
            a < x and x < b implies f(x) = c
            f(x) = c
            interval_contains(x, y, x) and f(x) = c
            exists(t: Real) {
                interval_contains(x, y, t) and f(t) = c
            }
        }
    }
}

/// On a nondegenerate subinterval of [a, b], the infimum of a bounded
/// function constant on (a, b) is at most c.
theorem interior_constant_inf_le_c(f: Real -> Real, a: Real, b: Real, c: Real, x: Real, y: Real, lb: Real) {
    a <= x and x < y and y <= b and
    (forall(t: Real) { a < t and t < b implies f(t) = c }) and
    (forall(t: Real) { interval_contains(a, b, t) implies lb <= f(t) })
    implies
    interval_inf(f, x, y) <= c
} by {
    if a <= x and x < y and y <= b and
       (forall(t: Real) { a < t and t < b implies f(t) = c }) and
       (forall(t: Real) { interval_contains(a, b, t) implies lb <= f(t) }) {
        lt_of_lt_of_lte(x, y, b)
        x < b
        lt_imp_lte(x, b)
        x <= b
        a <= x and x <= b
        interval_contains(a, b, x)
        lt_of_lte_of_lt(a, x, y)
        a < y
        lt_imp_lte(a, y)
        a <= y
        a <= y and y <= b
        interval_contains(a, b, y)
        forall(t0: Real) {
            if interval_contains(x, y, t0) {
                interval_contains_left(x, y, t0)
                x <= t0
                interval_contains_right(x, y, t0)
                t0 <= y
                interval_contains_mono(a, b, x, y, t0)
                interval_contains(a, b, t0)
                forall(t1: Real) {
                    interval_contains(a, b, t1) implies lb <= f(t1)
                }
                interval_contains(a, b, t0) implies lb <= f(t0)
                lb <= f(t0)
            }
        }
        interior_constant_cell_point(f, a, b, c, x, y)
        let t: Real satisfy {
            interval_contains(x, y, t) and f(t) = c
        }
        interval_set_contains_left(x, y)
        interval_set(x, y).contains(x)
        lt_imp_lte(x, y)
        x <= y
        interval_contains(x, y, x)
        interval_set(x, y).contains(x) = interval_contains(x, y, x)
        interval_set(x, y).contains(x)
        is_nonempty(interval_set(x, y))
        image_lower_bound(f, x, y, lb)
        is_set_lower_bound(interval_image(f, x, y), lb)
        exists(bb: Real) {
            is_set_lower_bound(interval_image(f, x, y), bb)
        }
        has_lower_bound(interval_image(f, x, y))
        is_nonempty(interval_set(x, y)) and has_lower_bound(interval_image(f, x, y))
        interval_inf_spec(f, x, y)
        is_set_infimum(interval_image(f, x, y), interval_inf(f, x, y))
        set_infimum_is_lower_bound(interval_image(f, x, y), interval_inf(f, x, y))
        is_set_lower_bound(interval_image(f, x, y), interval_inf(f, x, y))
        interval_set(x, y).contains(t) = interval_contains(x, y, t)
        interval_contains(x, y, t)
        interval_set(x, y).contains(t)
        maps_into_set_image(interval_set(x, y), f, t)
        function_image(f, interval_set(x, y)).contains(f(t))
        interval_image(f, x, y).contains(f(t))
        set_lower_bound_contains_le(interval_image(f, x, y), interval_inf(f, x, y), f(t))
        interval_inf(f, x, y) <= f(t)
        f(t) = c
        interval_inf(f, x, y) <= c
    }
}

/// On a nondegenerate subinterval of [a, b], the supremum of a bounded
/// function constant on (a, b) is at least c.
theorem interior_constant_c_le_sup(f: Real -> Real, a: Real, b: Real, c: Real, x: Real, y: Real, ub: Real) {
    a <= x and x < y and y <= b and
    (forall(t: Real) { a < t and t < b implies f(t) = c }) and
    (forall(t: Real) { interval_contains(a, b, t) implies f(t) <= ub })
    implies
    c <= interval_sup(f, x, y)
} by {
    if a <= x and x < y and y <= b and
       (forall(t: Real) { a < t and t < b implies f(t) = c }) and
       (forall(t: Real) { interval_contains(a, b, t) implies f(t) <= ub }) {
        lt_of_lt_of_lte(x, y, b)
        x < b
        lt_imp_lte(x, b)
        x <= b
        a <= x and x <= b
        interval_contains(a, b, x)
        lt_of_lte_of_lt(a, x, y)
        a < y
        lt_imp_lte(a, y)
        a <= y
        a <= y and y <= b
        interval_contains(a, b, y)
        forall(t0: Real) {
            if interval_contains(x, y, t0) {
                interval_contains_left(x, y, t0)
                x <= t0
                interval_contains_right(x, y, t0)
                t0 <= y
                interval_contains_mono(a, b, x, y, t0)
                interval_contains(a, b, t0)
                forall(t1: Real) {
                    interval_contains(a, b, t1) implies f(t1) <= ub
                }
                interval_contains(a, b, t0) implies f(t0) <= ub
                f(t0) <= ub
            }
        }
        interior_constant_cell_point(f, a, b, c, x, y)
        let t: Real satisfy {
            interval_contains(x, y, t) and f(t) = c
        }
        interval_set_contains_left(x, y)
        interval_set(x, y).contains(x)
        lt_imp_lte(x, y)
        x <= y
        interval_contains(x, y, x)
        interval_set(x, y).contains(x) = interval_contains(x, y, x)
        interval_set(x, y).contains(x)
        is_nonempty(interval_set(x, y))
        image_upper_bound(f, x, y, ub)
        is_set_upper_bound(interval_image(f, x, y), ub)
        exists(bb0: Real) {
            is_set_upper_bound(interval_image(f, x, y), bb0)
        }
        has_upper_bound(interval_image(f, x, y))
        is_nonempty(interval_set(x, y)) and has_upper_bound(interval_image(f, x, y))
        interval_sup_spec(f, x, y)
        is_set_supremum(interval_image(f, x, y), interval_sup(f, x, y))
        set_supremum_is_upper_bound(interval_image(f, x, y), interval_sup(f, x, y))
        is_set_upper_bound(interval_image(f, x, y), interval_sup(f, x, y))
        interval_set(x, y).contains(t) = interval_contains(x, y, t)
        interval_contains(x, y, t)
        interval_set(x, y).contains(t)
        maps_into_set_image(interval_set(x, y), f, t)
        function_image(f, interval_set(x, y)).contains(f(t))
        interval_image(f, x, y).contains(f(t))
        set_upper_bound_contains_le(interval_image(f, x, y), interval_sup(f, x, y), f(t))
        f(t) <= interval_sup(f, x, y)
        f(t) = c
        c <= interval_sup(f, x, y)
    }
}

/// The lower Darboux step of a bounded function constant on (a, b) is at most
/// c times the width of the subinterval.
theorem interior_constant_step_lower_le_c_width(f: Real -> Real, p: Nat -> Real, a: Real, b: Real, n: Nat, k: Nat, c: Real, lb: Real, ub: Real) {
    a <= b and is_partition(p, a, b, n) and k < n and
    (forall(t: Real) { a < t and t < b implies f(t) = c }) and
    (forall(t: Real) { interval_contains(a, b, t) implies lb <= f(t) }) and
    (forall(t: Real) { interval_contains(a, b, t) implies f(t) <= ub })
    implies
    partition_step_lower(f, p, k) <= c * diff_step(p, k)
} by {
    if a <= b and is_partition(p, a, b, n) and k < n and
       (forall(t: Real) { a < t and t < b implies f(t) = c }) and
       (forall(t: Real) { interval_contains(a, b, t) implies lb <= f(t) }) and
       (forall(t: Real) { interval_contains(a, b, t) implies f(t) <= ub }) {
        lt_imp_lte_suc(k, n)
        k + 1 <= n
        k <= k + 1
        partition_mono(p, a, b, n, k, k + 1)
        p(k) <= p(k + 1)
        partition_point_in_interval(p, a, b, n, k)
        interval_contains(a, b, p(k))
        partition_point_in_interval(p, a, b, n, k + 1)
        interval_contains(a, b, p(k + 1))
        interval_contains_left(a, b, p(k))
        a <= p(k)
        interval_contains_right(a, b, p(k + 1))
        p(k + 1) <= b
        if p(k) = p(k + 1) {
            diff_step(p, k) = p(k + 1) - p(k)
            p(k + 1) - p(k) = Real.0
            diff_step(p, k) = Real.0
            partition_step_lower(f, p, k) = interval_inf(f, p(k), p(k + 1)) * diff_step(p, k)
            interval_inf(f, p(k), p(k + 1)) * diff_step(p, k) = interval_inf(f, p(k), p(k + 1)) * Real.0
            interval_inf(f, p(k), p(k + 1)) * Real.0 = Real.0
            partition_step_lower(f, p, k) = Real.0
            c * diff_step(p, k) = c * Real.0
            c * Real.0 = Real.0
            c * diff_step(p, k) = Real.0
            partition_step_lower(f, p, k) <= c * diff_step(p, k)
        } else {
            not p(k) = p(k + 1)
            lt_of_lte_of_ne(p(k), p(k + 1))
            p(k) < p(k + 1)
            interior_constant_inf_le_c(f, a, b, c, p(k), p(k + 1), lb)
            interval_inf(f, p(k), p(k + 1)) <= c
            sub_nonneg(p(k), p(k + 1))
            Real.0 <= p(k + 1) - p(k)
            mul_le_mul_of_nonneg_right(interval_inf(f, p(k), p(k + 1)), c, p(k + 1) - p(k))
            interval_inf(f, p(k), p(k + 1)) * (p(k + 1) - p(k)) <= c * (p(k + 1) - p(k))
            partition_step_lower(f, p, k) = interval_inf(f, p(k), p(k + 1)) * diff_step(p, k)
            diff_step(p, k) = p(k + 1) - p(k)
            interval_inf(f, p(k), p(k + 1)) * diff_step(p, k) = interval_inf(f, p(k), p(k + 1)) * (p(k + 1) - p(k))
            c * diff_step(p, k) = c * (p(k + 1) - p(k))
            partition_step_lower(f, p, k) <= c * diff_step(p, k)
        }
    }
}

/// The upper Darboux step of a bounded function constant on (a, b) is at
/// least c times the width of the subinterval.
theorem interior_constant_step_upper_ge_c_width(f: Real -> Real, p: Nat -> Real, a: Real, b: Real, n: Nat, k: Nat, c: Real, lb: Real, ub: Real) {
    a <= b and is_partition(p, a, b, n) and k < n and
    (forall(t: Real) { a < t and t < b implies f(t) = c }) and
    (forall(t: Real) { interval_contains(a, b, t) implies lb <= f(t) }) and
    (forall(t: Real) { interval_contains(a, b, t) implies f(t) <= ub })
    implies
    c * diff_step(p, k) <= partition_step_upper(f, p, k)
} by {
    if a <= b and is_partition(p, a, b, n) and k < n and
       (forall(t: Real) { a < t and t < b implies f(t) = c }) and
       (forall(t: Real) { interval_contains(a, b, t) implies lb <= f(t) }) and
       (forall(t: Real) { interval_contains(a, b, t) implies f(t) <= ub }) {
        lt_imp_lte_suc(k, n)
        k + 1 <= n
        k <= k + 1
        partition_mono(p, a, b, n, k, k + 1)
        p(k) <= p(k + 1)
        partition_point_in_interval(p, a, b, n, k)
        interval_contains(a, b, p(k))
        partition_point_in_interval(p, a, b, n, k + 1)
        interval_contains(a, b, p(k + 1))
        interval_contains_left(a, b, p(k))
        a <= p(k)
        interval_contains_right(a, b, p(k + 1))
        p(k + 1) <= b
        if p(k) = p(k + 1) {
            diff_step(p, k) = p(k + 1) - p(k)
            p(k + 1) - p(k) = Real.0
            diff_step(p, k) = Real.0
            partition_step_upper(f, p, k) = interval_sup(f, p(k), p(k + 1)) * diff_step(p, k)
            interval_sup(f, p(k), p(k + 1)) * diff_step(p, k) = interval_sup(f, p(k), p(k + 1)) * Real.0
            interval_sup(f, p(k), p(k + 1)) * Real.0 = Real.0
            partition_step_upper(f, p, k) = Real.0
            c * diff_step(p, k) = c * Real.0
            c * Real.0 = Real.0
            c * diff_step(p, k) = Real.0
            c * diff_step(p, k) <= partition_step_upper(f, p, k)
        } else {
            not p(k) = p(k + 1)
            lt_of_lte_of_ne(p(k), p(k + 1))
            p(k) < p(k + 1)
            interior_constant_c_le_sup(f, a, b, c, p(k), p(k + 1), ub)
            c <= interval_sup(f, p(k), p(k + 1))
            sub_nonneg(p(k), p(k + 1))
            Real.0 <= p(k + 1) - p(k)
            mul_le_mul_of_nonneg_right(c, interval_sup(f, p(k), p(k + 1)), p(k + 1) - p(k))
            c * (p(k + 1) - p(k)) <= interval_sup(f, p(k), p(k + 1)) * (p(k + 1) - p(k))
            partition_step_upper(f, p, k) = interval_sup(f, p(k), p(k + 1)) * diff_step(p, k)
            diff_step(p, k) = p(k + 1) - p(k)
            interval_sup(f, p(k), p(k + 1)) * diff_step(p, k) = interval_sup(f, p(k), p(k + 1)) * (p(k + 1) - p(k))
            c * diff_step(p, k) = c * (p(k + 1) - p(k))
            c * diff_step(p, k) <= partition_step_upper(f, p, k)
        }
    }
}

/// Every lower Darboux sum of a bounded function constant on (a, b) is at
/// most c * (b - a).
theorem interior_constant_lower_sum_le_c_width(f: Real -> Real, p: Nat -> Real, a: Real, b: Real, n: Nat, c: Real, lb: Real, ub: Real) {
    a <= b and is_partition(p, a, b, n) and
    (forall(t: Real) { a < t and t < b implies f(t) = c }) and
    (forall(t: Real) { interval_contains(a, b, t) implies lb <= f(t) }) and
    (forall(t: Real) { interval_contains(a, b, t) implies f(t) <= ub })
    implies
    lower_sum(f, p, n) <= c * (b - a)
} by {
    if a <= b and is_partition(p, a, b, n) and
       (forall(t: Real) { a < t and t < b implies f(t) = c }) and
       (forall(t: Real) { interval_contains(a, b, t) implies lb <= f(t) }) and
       (forall(t: Real) { interval_contains(a, b, t) implies f(t) <= ub }) {
        forall(k: Nat) {
            if k < n {
                interior_constant_step_lower_le_c_width(f, p, a, b, n, k, c, lb, ub)
                partition_step_lower(f, p, k) <= scalar_diff_step(c, p, k)
            }
        }
        partial_lte(partition_step_lower(f, p), scalar_diff_step(c, p), n)
        partial(partition_step_lower(f, p), n) <= partial(scalar_diff_step(c, p), n)
        scalar_diff_step_eq_mul_fn(c, p)
        scalar_diff_step(c, p) = mul_fn(c, diff_step(p))
        partial(scalar_diff_step(c, p), n) = partial(mul_fn(c, diff_step(p)), n)
        partial_scalar_mul(c, diff_step(p), n)
        c * partial(diff_step(p), n) = partial(mul_fn(c, diff_step(p)), n)
        partial(scalar_diff_step(c, p), n) = c * partial(diff_step(p), n)
        telescope(p, n)
        partial(diff_step(p), n) = p(n) - p(Nat.0)
        partition_start(p, a, b, n)
        p(Nat.0) = a
        partition_end(p, a, b, n)
        p(n) = b
        p(n) - p(Nat.0) = b - a
        c * (p(n) - p(Nat.0)) = c * (b - a)
        partial(partition_step_lower(f, p), n) <= c * (b - a)
        lower_sum(f, p, n) <= c * (b - a)
    }
}

/// Every upper Darboux sum of a bounded function constant on (a, b) is at
/// least c * (b - a).
theorem interior_constant_c_width_le_upper_sum(f: Real -> Real, p: Nat -> Real, a: Real, b: Real, n: Nat, c: Real, lb: Real, ub: Real) {
    a <= b and is_partition(p, a, b, n) and
    (forall(t: Real) { a < t and t < b implies f(t) = c }) and
    (forall(t: Real) { interval_contains(a, b, t) implies lb <= f(t) }) and
    (forall(t: Real) { interval_contains(a, b, t) implies f(t) <= ub })
    implies
    c * (b - a) <= upper_sum(f, p, n)
} by {
    if a <= b and is_partition(p, a, b, n) and
       (forall(t: Real) { a < t and t < b implies f(t) = c }) and
       (forall(t: Real) { interval_contains(a, b, t) implies lb <= f(t) }) and
       (forall(t: Real) { interval_contains(a, b, t) implies f(t) <= ub }) {
        forall(k: Nat) {
            if k < n {
                interior_constant_step_upper_ge_c_width(f, p, a, b, n, k, c, lb, ub)
                scalar_diff_step(c, p, k) <= partition_step_upper(f, p, k)
            }
        }
        partial_lte(scalar_diff_step(c, p), partition_step_upper(f, p), n)
        partial(scalar_diff_step(c, p), n) <= partial(partition_step_upper(f, p), n)
        scalar_diff_step_eq_mul_fn(c, p)
        scalar_diff_step(c, p) = mul_fn(c, diff_step(p))
        partial(scalar_diff_step(c, p), n) = partial(mul_fn(c, diff_step(p)), n)
        partial_scalar_mul(c, diff_step(p), n)
        c * partial(diff_step(p), n) = partial(mul_fn(c, diff_step(p)), n)
        partial(scalar_diff_step(c, p), n) = c * partial(diff_step(p), n)
        telescope(p, n)
        partial(diff_step(p), n) = p(n) - p(Nat.0)
        partition_start(p, a, b, n)
        p(Nat.0) = a
        partition_end(p, a, b, n)
        p(n) = b
        p(n) - p(Nat.0) = b - a
        c * (p(n) - p(Nat.0)) = c * (b - a)
        c * (b - a) <= partial(partition_step_upper(f, p), n)
        c * (b - a) <= upper_sum(f, p, n)
    }
}

/// The difference of the upper and lower Darboux steps of a bounded function
/// over a subinterval of a partition of [a, b] is at most (ub - lb) times the
/// width of that subinterval.
theorem bounded_step_upper_minus_lower_le_width(f: Real -> Real, p: Nat -> Real, a: Real, b: Real, n: Nat, k: Nat, lb: Real, ub: Real) {
    a <= b and is_partition(p, a, b, n) and k < n and
    (forall(t: Real) { interval_contains(a, b, t) implies lb <= f(t) }) and
    (forall(t: Real) { interval_contains(a, b, t) implies f(t) <= ub })
    implies
    partition_step_upper(f, p, k) - partition_step_lower(f, p, k) <= (ub - lb) * diff_step(p, k)
} by {
    if a <= b and is_partition(p, a, b, n) and k < n and
       (forall(t: Real) { interval_contains(a, b, t) implies lb <= f(t) }) and
       (forall(t: Real) { interval_contains(a, b, t) implies f(t) <= ub }) {
        lt_imp_lte_suc(k, n)
        k + 1 <= n
        k <= k + 1
        partition_mono(p, a, b, n, k, k + 1)
        p(k) <= p(k + 1)
        partition_point_in_interval(p, a, b, n, k)
        interval_contains(a, b, p(k))
        partition_point_in_interval(p, a, b, n, k + 1)
        interval_contains(a, b, p(k + 1))
        forall(t: Real) {
            if interval_contains(p(k), p(k + 1), t) {
                interval_contains_left(p(k), p(k + 1), t)
                p(k) <= t
                interval_contains_right(p(k), p(k + 1), t)
                t <= p(k + 1)
                interval_contains_mono(a, b, p(k), p(k + 1), t)
                interval_contains(a, b, t)
                forall(t0: Real) {
                    interval_contains(a, b, t0) implies lb <= f(t0)
                }
                interval_contains(a, b, t) implies lb <= f(t)
                lb <= f(t)
            }
        }
        forall(t: Real) {
            if interval_contains(p(k), p(k + 1), t) {
                interval_contains_left(p(k), p(k + 1), t)
                p(k) <= t
                interval_contains_right(p(k), p(k + 1), t)
                t <= p(k + 1)
                interval_contains_mono(a, b, p(k), p(k + 1), t)
                interval_contains(a, b, t)
                forall(t1: Real) {
                    interval_contains(a, b, t1) implies f(t1) <= ub
                }
                interval_contains(a, b, t) implies f(t) <= ub
                f(t) <= ub
            }
        }
        interval_set_contains_left(p(k), p(k + 1))
        is_nonempty(interval_set(p(k), p(k + 1)))
        image_lower_bound(f, p(k), p(k + 1), lb)
        is_set_lower_bound(interval_image(f, p(k), p(k + 1)), lb)
        exists(bb: Real) {
            is_set_lower_bound(interval_image(f, p(k), p(k + 1)), bb)
        }
        has_lower_bound(interval_image(f, p(k), p(k + 1)))
        is_nonempty(interval_set(p(k), p(k + 1))) and has_lower_bound(interval_image(f, p(k), p(k + 1)))
        interval_inf_spec(f, p(k), p(k + 1))
        is_set_infimum(interval_image(f, p(k), p(k + 1)), interval_inf(f, p(k), p(k + 1)))
        set_infimum_is_lower_bound(interval_image(f, p(k), p(k + 1)), interval_inf(f, p(k), p(k + 1)))
        is_set_lower_bound(interval_image(f, p(k), p(k + 1)), interval_inf(f, p(k), p(k + 1)))
        image_upper_bound(f, p(k), p(k + 1), ub)
        is_set_upper_bound(interval_image(f, p(k), p(k + 1)), ub)
        exists(bb0: Real) {
            is_set_upper_bound(interval_image(f, p(k), p(k + 1)), bb0)
        }
        has_upper_bound(interval_image(f, p(k), p(k + 1)))
        is_nonempty(interval_set(p(k), p(k + 1))) and has_upper_bound(interval_image(f, p(k), p(k + 1)))
        interval_sup_spec(f, p(k), p(k + 1))
        is_set_supremum(interval_image(f, p(k), p(k + 1)), interval_sup(f, p(k), p(k + 1)))
        set_supremum_le_upper_bound(interval_image(f, p(k), p(k + 1)), interval_sup(f, p(k), p(k + 1)), ub)
        interval_sup(f, p(k), p(k + 1)) <= ub
        set_lower_bound_le_infimum(interval_image(f, p(k), p(k + 1)), interval_inf(f, p(k), p(k + 1)), lb)
        lb <= interval_inf(f, p(k), p(k + 1))
        neg_lte_flip(lb, interval_inf(f, p(k), p(k + 1)))
        -interval_inf(f, p(k), p(k + 1)) <= -lb
        add_le_add(interval_sup(f, p(k), p(k + 1)), ub, -interval_inf(f, p(k), p(k + 1)), -lb)
        interval_sup(f, p(k), p(k + 1)) + -interval_inf(f, p(k), p(k + 1)) <= ub + -lb
        interval_sup(f, p(k), p(k + 1)) - interval_inf(f, p(k), p(k + 1)) = interval_sup(f, p(k), p(k + 1)) + -interval_inf(f, p(k), p(k + 1))
        interval_sup(f, p(k), p(k + 1)) - interval_inf(f, p(k), p(k + 1)) <= ub + -lb
        ub - lb = ub + -lb
        interval_sup(f, p(k), p(k + 1)) - interval_inf(f, p(k), p(k + 1)) <= ub - lb
        sub_nonneg(p(k), p(k + 1))
        Real.0 <= p(k + 1) - p(k)
        mul_le_mul_of_nonneg_right(interval_sup(f, p(k), p(k + 1)) - interval_inf(f, p(k), p(k + 1)), ub - lb, p(k + 1) - p(k))
        (interval_sup(f, p(k), p(k + 1)) - interval_inf(f, p(k), p(k + 1))) * (p(k + 1) - p(k)) <= (ub - lb) * (p(k + 1) - p(k))
        partition_step_upper(f, p, k) - partition_step_lower(f, p, k) =
            (interval_sup(f, p(k), p(k + 1)) - interval_inf(f, p(k), p(k + 1))) * (p(k + 1) - p(k))
        partition_step_upper(f, p, k) - partition_step_lower(f, p, k) <= (ub - lb) * (p(k + 1) - p(k))
        (ub - lb) * (p(k + 1) - p(k)) = (ub - lb) * diff_step(p, k)
        partition_step_upper(f, p, k) - partition_step_lower(f, p, k) <= (ub - lb) * diff_step(p, k)
    }
}

/// On an interior cell of the uniform partition of [a, b], a function
/// constant on (a, b) has zero oscillation.
theorem uniform_interior_cell_constant(f: Real -> Real, a: Real, b: Real, m: Nat, i: Nat, c: Real) {
    a < b and Nat.1 <= i and i + 1 <= m and
    (forall(t: Real) { a < t and t < b implies f(t) = c })
    implies
    interval_inf(f, uniform_partition(a, b, m, i), uniform_partition(a, b, m, i + 1)) = c and
        interval_sup(f, uniform_partition(a, b, m, i), uniform_partition(a, b, m, i + 1)) = c
} by {
    if a < b and Nat.1 <= i and i + 1 <= m and (forall(t: Real) { a < t and t < b implies f(t) = c }) {
        lt_imp_lte(a, b)
        a <= b
        lt_imp_pos_sub(a, b)
        Real.0 < b - a
        from_nat_suc_pos_real(m)
        from_nat[Real](m.suc) > Real.0
        gt_zero_imp_pos(b - a)
        (b - a).is_positive
        gt_zero_imp_pos(from_nat[Real](m.suc))
        from_nat[Real](m.suc).is_positive
        div_pos_of_pos_pos(b - a, from_nat[Real](m.suc))
        (b - a) / from_nat[Real](m.suc) > Real.0
        gt_zero_imp_pos((b - a) / from_nat[Real](m.suc))
        ((b - a) / from_nat[Real](m.suc)).is_positive
        from_nat_lte_mono(Nat.1, i)
        from_nat[Real](Nat.1) <= from_nat[Real](i)
        from_nat[Real](Nat.1) = Real.1
        Real.1 <= from_nat[Real](i)
        Real.0 < Real.1
        lt_of_lt_of_lte(Real.0, Real.1, from_nat[Real](i))
        Real.0 < from_nat[Real](i)
        gt_zero_imp_pos(from_nat[Real](i))
        from_nat[Real](i).is_positive
        mul_pos_pos(from_nat[Real](i), (b - a) / from_nat[Real](m.suc))
        (from_nat[Real](i) * ((b - a) / from_nat[Real](m.suc))).is_positive
        pos_gt_zero(from_nat[Real](i) * ((b - a) / from_nat[Real](m.suc)))
        from_nat[Real](i) * ((b - a) / from_nat[Real](m.suc)) > Real.0
        uniform_partition(a, b, m, i) = a + from_nat[Real](i) * ((b - a) / from_nat[Real](m.suc))
        uniform_partition(a, b, m, i) - a = from_nat[Real](i) * ((b - a) / from_nat[Real](m.suc))
        uniform_partition(a, b, m, i) - a > Real.0
        lt_add_right(Real.0, uniform_partition(a, b, m, i) - a, a)
        Real.0 + a < uniform_partition(a, b, m, i) - a + a
        Real.0 + a = a
        uniform_partition(a, b, m, i) - a + a = uniform_partition(a, b, m, i)
        a < uniform_partition(a, b, m, i)
        i + 1 <= m
        m <= m.suc
        lte_trans(i + 1, m, m.suc)
        i + 1 <= m.suc
        uniform_partition_monotone(a, b, m, i + 1, m)
        uniform_partition(a, b, m, i + 1) <= uniform_partition(a, b, m, m)
        i <= i + 1
        lte_trans(i, i + 1, m.suc)
        i <= m.suc
        uniform_partition_monotone(a, b, m, i, i + 1)
        uniform_partition(a, b, m, i) <= uniform_partition(a, b, m, i + 1)
        interval_set_contains_left(uniform_partition(a, b, m, i), uniform_partition(a, b, m, i + 1))
        interval_set(uniform_partition(a, b, m, i), uniform_partition(a, b, m, i + 1)).contains(uniform_partition(a, b, m, i))
        is_nonempty(interval_set(uniform_partition(a, b, m, i), uniform_partition(a, b, m, i + 1)))
        forall(t: Real) {
            if interval_contains(uniform_partition(a, b, m, i), uniform_partition(a, b, m, i + 1), t) {
                interval_contains_left(uniform_partition(a, b, m, i), uniform_partition(a, b, m, i + 1), t)
                uniform_partition(a, b, m, i) <= t
                lt_of_lt_of_lte(a, uniform_partition(a, b, m, i), t)
                a < t
                interval_contains_right(uniform_partition(a, b, m, i), uniform_partition(a, b, m, i + 1), t)
                t <= uniform_partition(a, b, m, i + 1)
                uniform_partition(a, b, m, i + 1) <= uniform_partition(a, b, m, m)
                lte_trans[Real](t, uniform_partition(a, b, m, i + 1), uniform_partition(a, b, m, m))
                t <= uniform_partition(a, b, m, m)
                uniform_partition_width(a, b, m, m)
                uniform_partition(a, b, m, m.suc) - uniform_partition(a, b, m, m) = (b - a) / from_nat[Real](m.suc)
                (b - a) / from_nat[Real](m.suc) > Real.0
                uniform_partition(a, b, m, m.suc) - uniform_partition(a, b, m, m) > Real.0
                lt_add_right(Real.0, uniform_partition(a, b, m, m.suc) - uniform_partition(a, b, m, m), uniform_partition(a, b, m, m))
                Real.0 + uniform_partition(a, b, m, m) < uniform_partition(a, b, m, m.suc) - uniform_partition(a, b, m, m) + uniform_partition(a, b, m, m)
                Real.0 + uniform_partition(a, b, m, m) = uniform_partition(a, b, m, m)
                uniform_partition(a, b, m, m.suc) - uniform_partition(a, b, m, m) + uniform_partition(a, b, m, m) = uniform_partition(a, b, m, m.suc)
                uniform_partition(a, b, m, m) < uniform_partition(a, b, m, m.suc)
                uniform_partition_end(a, b, m)
                uniform_partition(a, b, m, m.suc) = b
                uniform_partition(a, b, m, m) < b
                lt_of_lte_of_lt(t, uniform_partition(a, b, m, m), b)
                t < b
                a < t and t < b
                forall(t0: Real) {
                    a < t0 and t0 < b implies f(t0) = c
                }
                a < t and t < b implies f(t) = c
                f(t) = c
            }
        }
        interval_inf_const(f, c, uniform_partition(a, b, m, i), uniform_partition(a, b, m, i + 1))
        interval_inf(f, uniform_partition(a, b, m, i), uniform_partition(a, b, m, i + 1)) = c
        interval_sup_const(f, c, uniform_partition(a, b, m, i), uniform_partition(a, b, m, i + 1))
        interval_sup(f, uniform_partition(a, b, m, i), uniform_partition(a, b, m, i + 1)) = c
        interval_inf(f, uniform_partition(a, b, m, i), uniform_partition(a, b, m, i + 1)) = c and interval_sup(f, uniform_partition(a, b, m, i), uniform_partition(a, b, m, i + 1)) = c
    }
}

/// The difference of the upper and lower Darboux sums over the uniform
/// partition of [a, b] is at most twice (ub - lb) times the mesh, for a
/// bounded function constant on (a, b).
theorem uniform_interior_constant_diffs_bound(f: Real -> Real, a: Real, b: Real, m: Nat, c: Real, lb: Real, ub: Real) {
    a < b and
    (forall(t: Real) { a < t and t < b implies f(t) = c }) and
    (forall(t: Real) { interval_contains(a, b, t) implies lb <= f(t) }) and
    (forall(t: Real) { interval_contains(a, b, t) implies f(t) <= ub })
    implies
    upper_sum(f, uniform_partition(a, b, m), m.suc) -
        lower_sum(f, uniform_partition(a, b, m), m.suc) <=
        two * (ub - lb) * ((b - a) / from_nat[Real](m.suc))
} by {
    if a < b and
       (forall(t: Real) { a < t and t < b implies f(t) = c }) and
       (forall(t: Real) { interval_contains(a, b, t) implies lb <= f(t) }) and
       (forall(t: Real) { interval_contains(a, b, t) implies f(t) <= ub }) {
        lt_imp_lte(a, b)
        a <= b
        uniform_partition_is_partition(a, b, m)
        is_partition(uniform_partition(a, b, m), a, b, m.suc)
        forall(i: Nat) {
            if i < m.suc {
                bounded_step_upper_minus_lower_le_width(f, uniform_partition(a, b, m), a, b, m.suc, i, lb, ub)
                partition_step_upper(f, uniform_partition(a, b, m), i) - partition_step_lower(f, uniform_partition(a, b, m), i) <= (ub - lb) * diff_step(uniform_partition(a, b, m), i)
                uniform_partition_width(a, b, m, i)
                uniform_partition(a, b, m, i + 1) - uniform_partition(a, b, m, i) = (b - a) / from_nat[Real](m.suc)
                diff_step(uniform_partition(a, b, m), i) = uniform_partition(a, b, m, i + 1) - uniform_partition(a, b, m, i)
                partition_step_upper(f, uniform_partition(a, b, m), i) - partition_step_lower(f, uniform_partition(a, b, m), i) <= (ub - lb) * ((b - a) / from_nat[Real](m.suc))
            }
        }
        forall(i: Nat) {
            if Nat.1 <= i and i + 1 <= m {
                uniform_interior_cell_constant(f, a, b, m, i, c)
                interval_inf(f, uniform_partition(a, b, m, i), uniform_partition(a, b, m, i + 1)) = c and interval_sup(f, uniform_partition(a, b, m, i), uniform_partition(a, b, m, i + 1)) = c
                interval_inf(f, uniform_partition(a, b, m, i), uniform_partition(a, b, m, i + 1)) = c
                interval_sup(f, uniform_partition(a, b, m, i), uniform_partition(a, b, m, i + 1)) = c
                partition_step_upper(f, uniform_partition(a, b, m), i) = interval_sup(f, uniform_partition(a, b, m, i), uniform_partition(a, b, m, i + 1)) * diff_step(uniform_partition(a, b, m), i)
                partition_step_lower(f, uniform_partition(a, b, m), i) = interval_inf(f, uniform_partition(a, b, m, i), uniform_partition(a, b, m, i + 1)) * diff_step(uniform_partition(a, b, m), i)
                partition_step_upper(f, uniform_partition(a, b, m), i) = c * diff_step(uniform_partition(a, b, m), i)
                partition_step_lower(f, uniform_partition(a, b, m), i) = c * diff_step(uniform_partition(a, b, m), i)
                partition_step_upper(f, uniform_partition(a, b, m), i) - partition_step_lower(f, uniform_partition(a, b, m), i) = c * diff_step(uniform_partition(a, b, m), i) - c * diff_step(uniform_partition(a, b, m), i)
                c * diff_step(uniform_partition(a, b, m), i) - c * diff_step(uniform_partition(a, b, m), i) = Real.0
                partition_step_upper(f, uniform_partition(a, b, m), i) - partition_step_lower(f, uniform_partition(a, b, m), i) = Real.0
            }
        }
        forall(i: Nat) {
            i < m.suc implies partition_step_upper(f, uniform_partition(a, b, m), i) - partition_step_lower(f, uniform_partition(a, b, m), i) <= (ub - lb) * ((b - a) / from_nat[Real](m.suc))
        }
        interval_contains(a, b, a) = a <= a and a <= b
        a <= a
        a <= b
        a <= a and a <= b
        interval_contains(a, b, a)
        forall(t: Real) {
            interval_contains(a, b, t) implies lb <= f(t)
        }
        interval_contains(a, b, a) implies lb <= f(a)
        lb <= f(a)
        forall(t0: Real) {
            interval_contains(a, b, t0) implies f(t0) <= ub
        }
        interval_contains(a, b, a) implies f(a) <= ub
        f(a) <= ub
        lte_trans[Real](lb, f(a), ub)
        lb <= ub
        sub_nonneg(lb, ub)
        Real.0 <= ub - lb
        add_le_add_right[Real](Real.0, ub - lb, ub - lb)
        Real.0 + (ub - lb) <= (ub - lb) + (ub - lb)
        Real.0 + (ub - lb) = ub - lb
        (ub - lb) + (ub - lb) = two * (ub - lb)
        ub - lb <= two * (ub - lb)
        sub_nonneg(a, b)
        Real.0 <= b - a
        from_nat_suc_pos_real(m)
        from_nat[Real](m.suc) > Real.0
        div_nonneg_pos_denom(b - a, from_nat[Real](m.suc))
        Real.0 <= (b - a) / from_nat[Real](m.suc)
        mul_le_mul_of_nonneg_right(ub - lb, two * (ub - lb), (b - a) / from_nat[Real](m.suc))
        (ub - lb) * ((b - a) / from_nat[Real](m.suc)) <= (two * (ub - lb)) * ((b - a) / from_nat[Real](m.suc))
        (two * (ub - lb)) * ((b - a) / from_nat[Real](m.suc)) = two * (ub - lb) * ((b - a) / from_nat[Real](m.suc))
        (ub - lb) * ((b - a) / from_nat[Real](m.suc)) <= two * (ub - lb) * ((b - a) / from_nat[Real](m.suc))
        if m = Nat.0 {
            partial_sub_seq(partition_step_upper(f, uniform_partition(a, b, m)), partition_step_lower(f, uniform_partition(a, b, m)), m.suc)
            partial(sub_seq(partition_step_upper(f, uniform_partition(a, b, m)), partition_step_lower(f, uniform_partition(a, b, m))), m.suc) = partial(partition_step_upper(f, uniform_partition(a, b, m)), m.suc) - partial(partition_step_lower(f, uniform_partition(a, b, m)), m.suc)
            upper_sum(f, uniform_partition(a, b, m), m.suc) = partial(partition_step_upper(f, uniform_partition(a, b, m)), m.suc)
            lower_sum(f, uniform_partition(a, b, m), m.suc) = partial(partition_step_lower(f, uniform_partition(a, b, m)), m.suc)
            partial_one(sub_seq(partition_step_upper(f, uniform_partition(a, b, m)), partition_step_lower(f, uniform_partition(a, b, m))))
            partial(sub_seq(partition_step_upper(f, uniform_partition(a, b, m)), partition_step_lower(f, uniform_partition(a, b, m))), Nat.1) = sub_seq(partition_step_upper(f, uniform_partition(a, b, m)), partition_step_lower(f, uniform_partition(a, b, m)), Nat.0)
            m.suc = Nat.1
            partial(sub_seq(partition_step_upper(f, uniform_partition(a, b, m)), partition_step_lower(f, uniform_partition(a, b, m))), m.suc) = sub_seq(partition_step_upper(f, uniform_partition(a, b, m)), partition_step_lower(f, uniform_partition(a, b, m)), Nat.0)
            sub_seq(partition_step_upper(f, uniform_partition(a, b, m)), partition_step_lower(f, uniform_partition(a, b, m)), Nat.0) = partition_step_upper(f, uniform_partition(a, b, m), Nat.0) - partition_step_lower(f, uniform_partition(a, b, m), Nat.0)
            upper_sum(f, uniform_partition(a, b, m), m.suc) - lower_sum(f, uniform_partition(a, b, m), m.suc) = partition_step_upper(f, uniform_partition(a, b, m), Nat.0) - partition_step_lower(f, uniform_partition(a, b, m), Nat.0)
            Nat.0 < m.suc
            forall(i: Nat) {
                i < m.suc implies partition_step_upper(f, uniform_partition(a, b, m), i) - partition_step_lower(f, uniform_partition(a, b, m), i) <= (ub - lb) * ((b - a) / from_nat[Real](m.suc))
            }
            Nat.0 < m.suc implies partition_step_upper(f, uniform_partition(a, b, m), Nat.0) - partition_step_lower(f, uniform_partition(a, b, m), Nat.0) <= (ub - lb) * ((b - a) / from_nat[Real](m.suc))
            partition_step_upper(f, uniform_partition(a, b, m), Nat.0) - partition_step_lower(f, uniform_partition(a, b, m), Nat.0) <= (ub - lb) * ((b - a) / from_nat[Real](m.suc))
            upper_sum(f, uniform_partition(a, b, m), m.suc) - lower_sum(f, uniform_partition(a, b, m), m.suc) <= (ub - lb) * ((b - a) / from_nat[Real](m.suc))
            lte_trans[Real](upper_sum(f, uniform_partition(a, b, m), m.suc) - lower_sum(f, uniform_partition(a, b, m), m.suc), (ub - lb) * ((b - a) / from_nat[Real](m.suc)), two * (ub - lb) * ((b - a) / from_nat[Real](m.suc)))
            upper_sum(f, uniform_partition(a, b, m), m.suc) - lower_sum(f, uniform_partition(a, b, m), m.suc) <= two * (ub - lb) * ((b - a) / from_nat[Real](m.suc))
        } else {
            not m = Nat.0
            pos_of_ne_zero(m)
            Nat.0 < m
            lt_imp_lte_suc(Nat.0, m)
            Nat.1 <= m
            partial_split_last(sub_seq(partition_step_upper(f, uniform_partition(a, b, m)), partition_step_lower(f, uniform_partition(a, b, m))), m)
            partial(sub_seq(partition_step_upper(f, uniform_partition(a, b, m)), partition_step_lower(f, uniform_partition(a, b, m))), m.suc) = partial(sub_seq(partition_step_upper(f, uniform_partition(a, b, m)), partition_step_lower(f, uniform_partition(a, b, m))), m) + sub_seq(partition_step_upper(f, uniform_partition(a, b, m)), partition_step_lower(f, uniform_partition(a, b, m)), m)
            partial_drop_first(sub_seq(partition_step_upper(f, uniform_partition(a, b, m)), partition_step_lower(f, uniform_partition(a, b, m))), m)
            partial(sub_seq(partition_step_upper(f, uniform_partition(a, b, m)), partition_step_lower(f, uniform_partition(a, b, m))), m) = sub_seq(partition_step_upper(f, uniform_partition(a, b, m)), partition_step_lower(f, uniform_partition(a, b, m)), Nat.0) + partial(compose(sub_seq(partition_step_upper(f, uniform_partition(a, b, m)), partition_step_lower(f, uniform_partition(a, b, m))), Nat.suc), m - 1)
            sub_seq(partition_step_upper(f, uniform_partition(a, b, m)), partition_step_lower(f, uniform_partition(a, b, m)), Nat.0) = partition_step_upper(f, uniform_partition(a, b, m), Nat.0) - partition_step_lower(f, uniform_partition(a, b, m), Nat.0)
            Nat.0 < m.suc
            forall(i: Nat) {
                i < m.suc implies partition_step_upper(f, uniform_partition(a, b, m), i) - partition_step_lower(f, uniform_partition(a, b, m), i) <= (ub - lb) * ((b - a) / from_nat[Real](m.suc))
            }
            Nat.0 < m.suc implies partition_step_upper(f, uniform_partition(a, b, m), Nat.0) - partition_step_lower(f, uniform_partition(a, b, m), Nat.0) <= (ub - lb) * ((b - a) / from_nat[Real](m.suc))
            partition_step_upper(f, uniform_partition(a, b, m), Nat.0) - partition_step_lower(f, uniform_partition(a, b, m), Nat.0) <= (ub - lb) * ((b - a) / from_nat[Real](m.suc))
            sub_seq(partition_step_upper(f, uniform_partition(a, b, m)), partition_step_lower(f, uniform_partition(a, b, m)), Nat.0) <= (ub - lb) * ((b - a) / from_nat[Real](m.suc))
            sub_seq(partition_step_upper(f, uniform_partition(a, b, m)), partition_step_lower(f, uniform_partition(a, b, m)), m) = partition_step_upper(f, uniform_partition(a, b, m), m) - partition_step_lower(f, uniform_partition(a, b, m), m)
            m < m.suc
            forall(i: Nat) {
                i < m.suc implies partition_step_upper(f, uniform_partition(a, b, m), i) - partition_step_lower(f, uniform_partition(a, b, m), i) <= (ub - lb) * ((b - a) / from_nat[Real](m.suc))
            }
            m < m.suc implies partition_step_upper(f, uniform_partition(a, b, m), m) - partition_step_lower(f, uniform_partition(a, b, m), m) <= (ub - lb) * ((b - a) / from_nat[Real](m.suc))
            partition_step_upper(f, uniform_partition(a, b, m), m) - partition_step_lower(f, uniform_partition(a, b, m), m) <= (ub - lb) * ((b - a) / from_nat[Real](m.suc))
            sub_seq(partition_step_upper(f, uniform_partition(a, b, m)), partition_step_lower(f, uniform_partition(a, b, m)), m) <= (ub - lb) * ((b - a) / from_nat[Real](m.suc))
            forall(k: Nat) {
                if k < m - 1 {
                    lt_imp_lte_suc(k, m - 1)
                    k + 1 <= m - 1
                    lte_suc_suc(Nat.0, k)
                    Nat.1 <= k + 1
                    lte_add_right(Nat.1, k + 1, m - 1)
                    k + 1 + 1 <= m - 1 + 1
                    add_sub(m, Nat.1)
                    m - 1 + 1 = m
                    k + 1 + 1 <= m
                    forall(i: Nat) {
                        if Nat.1 <= i and i + 1 <= m {
                            partition_step_upper(f, uniform_partition(a, b, m), i) - partition_step_lower(f, uniform_partition(a, b, m), i) = Real.0
                        }
                    }
                    Nat.1 <= k + 1 and k + 1 + 1 <= m
                    partition_step_upper(f, uniform_partition(a, b, m), k + 1) - partition_step_lower(f, uniform_partition(a, b, m), k + 1) = Real.0
                    compose(sub_seq(partition_step_upper(f, uniform_partition(a, b, m)), partition_step_lower(f, uniform_partition(a, b, m))), Nat.suc, k) = sub_seq(partition_step_upper(f, uniform_partition(a, b, m)), partition_step_lower(f, uniform_partition(a, b, m)), k + 1)
                    sub_seq(partition_step_upper(f, uniform_partition(a, b, m)), partition_step_lower(f, uniform_partition(a, b, m)), k + 1) = partition_step_upper(f, uniform_partition(a, b, m), k + 1) - partition_step_lower(f, uniform_partition(a, b, m), k + 1)
                    compose(sub_seq(partition_step_upper(f, uniform_partition(a, b, m)), partition_step_lower(f, uniform_partition(a, b, m))), Nat.suc, k) = Real.0
                }
            }
            partial_const(compose(sub_seq(partition_step_upper(f, uniform_partition(a, b, m)), partition_step_lower(f, uniform_partition(a, b, m))), Nat.suc), m - 1, Real.0)
            partial(compose(sub_seq(partition_step_upper(f, uniform_partition(a, b, m)), partition_step_lower(f, uniform_partition(a, b, m))), Nat.suc), m - 1) = from_nat[Real](m - 1) * Real.0
            from_nat[Real](m - 1) * Real.0 = Real.0
            partial(compose(sub_seq(partition_step_upper(f, uniform_partition(a, b, m)), partition_step_lower(f, uniform_partition(a, b, m))), Nat.suc), m - 1) = Real.0
            partial(sub_seq(partition_step_upper(f, uniform_partition(a, b, m)), partition_step_lower(f, uniform_partition(a, b, m))), m.suc) = sub_seq(partition_step_upper(f, uniform_partition(a, b, m)), partition_step_lower(f, uniform_partition(a, b, m)), Nat.0) + Real.0 + sub_seq(partition_step_upper(f, uniform_partition(a, b, m)), partition_step_lower(f, uniform_partition(a, b, m)), m)
            sub_seq(partition_step_upper(f, uniform_partition(a, b, m)), partition_step_lower(f, uniform_partition(a, b, m)), Nat.0) + Real.0 + sub_seq(partition_step_upper(f, uniform_partition(a, b, m)), partition_step_lower(f, uniform_partition(a, b, m)), m) = sub_seq(partition_step_upper(f, uniform_partition(a, b, m)), partition_step_lower(f, uniform_partition(a, b, m)), Nat.0) + sub_seq(partition_step_upper(f, uniform_partition(a, b, m)), partition_step_lower(f, uniform_partition(a, b, m)), m)
            add_le_add(sub_seq(partition_step_upper(f, uniform_partition(a, b, m)), partition_step_lower(f, uniform_partition(a, b, m)), Nat.0), (ub - lb) * ((b - a) / from_nat[Real](m.suc)), sub_seq(partition_step_upper(f, uniform_partition(a, b, m)), partition_step_lower(f, uniform_partition(a, b, m)), m), (ub - lb) * ((b - a) / from_nat[Real](m.suc)))
            sub_seq(partition_step_upper(f, uniform_partition(a, b, m)), partition_step_lower(f, uniform_partition(a, b, m)), Nat.0) + sub_seq(partition_step_upper(f, uniform_partition(a, b, m)), partition_step_lower(f, uniform_partition(a, b, m)), m) <= (ub - lb) * ((b - a) / from_nat[Real](m.suc)) + (ub - lb) * ((b - a) / from_nat[Real](m.suc))
            (ub - lb) * ((b - a) / from_nat[Real](m.suc)) + (ub - lb) * ((b - a) / from_nat[Real](m.suc)) = two * (ub - lb) * ((b - a) / from_nat[Real](m.suc))
            sub_seq(partition_step_upper(f, uniform_partition(a, b, m)), partition_step_lower(f, uniform_partition(a, b, m)), Nat.0) + sub_seq(partition_step_upper(f, uniform_partition(a, b, m)), partition_step_lower(f, uniform_partition(a, b, m)), m) <= two * (ub - lb) * ((b - a) / from_nat[Real](m.suc))
            partial(sub_seq(partition_step_upper(f, uniform_partition(a, b, m)), partition_step_lower(f, uniform_partition(a, b, m))), m.suc) <= two * (ub - lb) * ((b - a) / from_nat[Real](m.suc))
            partial_sub_seq(partition_step_upper(f, uniform_partition(a, b, m)), partition_step_lower(f, uniform_partition(a, b, m)), m.suc)
            partial(sub_seq(partition_step_upper(f, uniform_partition(a, b, m)), partition_step_lower(f, uniform_partition(a, b, m))), m.suc) = partial(partition_step_upper(f, uniform_partition(a, b, m)), m.suc) - partial(partition_step_lower(f, uniform_partition(a, b, m)), m.suc)
            upper_sum(f, uniform_partition(a, b, m), m.suc) = partial(partition_step_upper(f, uniform_partition(a, b, m)), m.suc)
            lower_sum(f, uniform_partition(a, b, m), m.suc) = partial(partition_step_lower(f, uniform_partition(a, b, m)), m.suc)
            upper_sum(f, uniform_partition(a, b, m), m.suc) - lower_sum(f, uniform_partition(a, b, m), m.suc) <= two * (ub - lb) * ((b - a) / from_nat[Real](m.suc))
        }
    }
}

/// When b = a, the difference of the upper and lower Darboux sums over the
/// trivial partition of [a, b] is zero.
theorem trivial_partition_diffs_zero(f: Real -> Real, a: Real, b: Real) {
    b = a implies
    upper_sum(f, trivial_partition(a, b), Nat.1) - lower_sum(f, trivial_partition(a, b), Nat.1) = Real.0
} by {
    if b = a {
        partial_one(partition_step_upper(f, trivial_partition(a, b)))
        partial(partition_step_upper(f, trivial_partition(a, b)), Nat.1) = partition_step_upper(f, trivial_partition(a, b), Nat.0)
        partition_step_upper(f, trivial_partition(a, b), Nat.0) = interval_sup(f, trivial_partition(a, b, Nat.0), trivial_partition(a, b, Nat.1)) * diff_step(trivial_partition(a, b), Nat.0)
        trivial_partition_zero(a, b)
        trivial_partition(a, b, Nat.0) = a
        trivial_partition_one(a, b)
        trivial_partition(a, b, Nat.1) = b
        interval_sup(f, trivial_partition(a, b, Nat.0), trivial_partition(a, b, Nat.1)) = interval_sup(f, a, b)
        partition_step_upper(f, trivial_partition(a, b), Nat.0) = interval_sup(f, a, b) * diff_step(trivial_partition(a, b), Nat.0)
        diff_step(trivial_partition(a, b), Nat.0) = trivial_partition(a, b, Nat.1) - trivial_partition(a, b, Nat.0)
        trivial_partition(a, b, Nat.1) - trivial_partition(a, b, Nat.0) = b - a
        b - a = a - a
        a - a = Real.0
        diff_step(trivial_partition(a, b), Nat.0) = Real.0
        interval_sup(f, a, b) * diff_step(trivial_partition(a, b), Nat.0) = interval_sup(f, a, b) * Real.0
        interval_sup(f, a, b) * Real.0 = Real.0
        partition_step_upper(f, trivial_partition(a, b), Nat.0) = Real.0
        partial(partition_step_upper(f, trivial_partition(a, b)), Nat.1) = Real.0
        upper_sum(f, trivial_partition(a, b), Nat.1) = partial(partition_step_upper(f, trivial_partition(a, b)), Nat.1)
        upper_sum(f, trivial_partition(a, b), Nat.1) = Real.0
        partial_one(partition_step_lower(f, trivial_partition(a, b)))
        partial(partition_step_lower(f, trivial_partition(a, b)), Nat.1) = partition_step_lower(f, trivial_partition(a, b), Nat.0)
        partition_step_lower(f, trivial_partition(a, b), Nat.0) = interval_inf(f, trivial_partition(a, b, Nat.0), trivial_partition(a, b, Nat.1)) * diff_step(trivial_partition(a, b), Nat.0)
        interval_inf(f, trivial_partition(a, b, Nat.0), trivial_partition(a, b, Nat.1)) = interval_inf(f, a, b)
        partition_step_lower(f, trivial_partition(a, b), Nat.0) = interval_inf(f, a, b) * diff_step(trivial_partition(a, b), Nat.0)
        interval_inf(f, a, b) * diff_step(trivial_partition(a, b), Nat.0) = interval_inf(f, a, b) * Real.0
        interval_inf(f, a, b) * Real.0 = Real.0
        partition_step_lower(f, trivial_partition(a, b), Nat.0) = Real.0
        partial(partition_step_lower(f, trivial_partition(a, b)), Nat.1) = Real.0
        lower_sum(f, trivial_partition(a, b), Nat.1) = partial(partition_step_lower(f, trivial_partition(a, b)), Nat.1)
        lower_sum(f, trivial_partition(a, b), Nat.1) = Real.0
        upper_sum(f, trivial_partition(a, b), Nat.1) - lower_sum(f, trivial_partition(a, b), Nat.1) = Real.0 - Real.0
        Real.0 - Real.0 = Real.0
        upper_sum(f, trivial_partition(a, b), Nat.1) - lower_sum(f, trivial_partition(a, b), Nat.1) = Real.0
    }
}

/// A bounded function constant on the open interval (a, b) satisfies the
/// Darboux eps-condition on [a, b].
theorem interior_constant_darboux_condition(f: Real -> Real, a: Real, b: Real, c: Real, lb: Real, ub: Real) {
    a <= b and
    (forall(t: Real) { a < t and t < b implies f(t) = c }) and
    (forall(t: Real) { interval_contains(a, b, t) implies lb <= f(t) }) and
    (forall(t: Real) { interval_contains(a, b, t) implies f(t) <= ub })
    implies
    darboux_condition(f, a, b)
} by {
    if a <= b and
       (forall(t: Real) { a < t and t < b implies f(t) = c }) and
       (forall(t: Real) { interval_contains(a, b, t) implies lb <= f(t) }) and
       (forall(t: Real) { interval_contains(a, b, t) implies f(t) <= ub }) {
        forall(eps: Real) {
            if eps.is_positive {
                if b = a {
                    trivial_partition_is_partition(a, b)
                    is_partition(trivial_partition(a, b), a, b, Nat.1)
                    trivial_partition_diffs_zero(f, a, b)
                    upper_sum(f, trivial_partition(a, b), Nat.1) - lower_sum(f, trivial_partition(a, b), Nat.1) = Real.0
                    pos_gt_zero(eps)
                    Real.0 < eps
                    upper_sum(f, trivial_partition(a, b), Nat.1) - lower_sum(f, trivial_partition(a, b), Nat.1) < eps
                    is_partition(trivial_partition(a, b), a, b, Nat.1) and upper_sum(f, trivial_partition(a, b), Nat.1) - lower_sum(f, trivial_partition(a, b), Nat.1) < eps
                    exists(p: Nat -> Real, n0: Nat) {
                        is_partition(p, a, b, n0) and upper_sum(f, p, n0) - lower_sum(f, p, n0) < eps
                    }
                } else {
                    not b = a
                    a != b
                    lt_of_lte_of_ne(a, b)
                    a < b
                    exists_n_frac_lt(two * (ub - lb) * (b - a), eps)
                    let m: Nat satisfy {
                        (two * (ub - lb) * (b - a)) / from_nat[Real](m.suc) < eps
                    }
                    (two * (ub - lb) * (b - a)) / from_nat[Real](m.suc) < eps
                    uniform_interior_constant_diffs_bound(f, a, b, m, c, lb, ub)
                    upper_sum(f, uniform_partition(a, b, m), m.suc) - lower_sum(f, uniform_partition(a, b, m), m.suc) <= two * (ub - lb) * ((b - a) / from_nat[Real](m.suc))
                    two * (ub - lb) * ((b - a) / from_nat[Real](m.suc)) = (two * (ub - lb)) * ((b - a) / from_nat[Real](m.suc))
                    mul_frac_right(b - a, from_nat[Real](m.suc), two * (ub - lb))
                    ((b - a) / from_nat[Real](m.suc)) * (two * (ub - lb)) = ((b - a) * (two * (ub - lb))) / from_nat[Real](m.suc)
                    (two * (ub - lb)) * ((b - a) / from_nat[Real](m.suc)) = ((b - a) * (two * (ub - lb))) / from_nat[Real](m.suc)
                    (b - a) * (two * (ub - lb)) = (two * (ub - lb)) * (b - a)
                    ((b - a) * (two * (ub - lb))) / from_nat[Real](m.suc) = ((two * (ub - lb)) * (b - a)) / from_nat[Real](m.suc)
                    two * (ub - lb) * ((b - a) / from_nat[Real](m.suc)) = ((two * (ub - lb)) * (b - a)) / from_nat[Real](m.suc)
                    (two * (ub - lb)) * (b - a) = two * (ub - lb) * (b - a)
                    two * (ub - lb) * ((b - a) / from_nat[Real](m.suc)) = (two * (ub - lb) * (b - a)) / from_nat[Real](m.suc)
                    (two * (ub - lb) * (b - a)) / from_nat[Real](m.suc) < eps
                    lt_of_lte_of_lt(upper_sum(f, uniform_partition(a, b, m), m.suc) - lower_sum(f, uniform_partition(a, b, m), m.suc), (two * (ub - lb) * (b - a)) / from_nat[Real](m.suc), eps)
                    upper_sum(f, uniform_partition(a, b, m), m.suc) - lower_sum(f, uniform_partition(a, b, m), m.suc) < eps
                    uniform_partition_is_partition(a, b, m)
                    is_partition(uniform_partition(a, b, m), a, b, m.suc)
                    is_partition(uniform_partition(a, b, m), a, b, m.suc) and upper_sum(f, uniform_partition(a, b, m), m.suc) - lower_sum(f, uniform_partition(a, b, m), m.suc) < eps
                    exists(p: Nat -> Real, n0: Nat) {
                        is_partition(p, a, b, n0) and upper_sum(f, p, n0) - lower_sum(f, p, n0) < eps
                    }
                }
            }
        }
        forall(eps0: Real) {
            eps0.is_positive implies darboux_partition_exists(f, a, b, eps0)
        }
        darboux_condition(f, a, b)
    }
}

/// A bounded function constant on the open interval (a, b) is integrable on
/// [a, b].
theorem interior_constant_integrable(f: Real -> Real, a: Real, b: Real, c: Real, lb: Real, ub: Real) {
    a <= b and
    (forall(t: Real) { a < t and t < b implies f(t) = c }) and
    (forall(t: Real) { interval_contains(a, b, t) implies lb <= f(t) }) and
    (forall(t: Real) { interval_contains(a, b, t) implies f(t) <= ub })
    implies
    is_integrable(f, a, b)
} by {
    if a <= b and
       (forall(t: Real) { a < t and t < b implies f(t) = c }) and
       (forall(t: Real) { interval_contains(a, b, t) implies lb <= f(t) }) and
       (forall(t: Real) { interval_contains(a, b, t) implies f(t) <= ub }) {
        lower_sum_set_sup_exists_of_bounded(f, a, b, lb, ub)
        let l: Real satisfy {
            is_set_supremum(lower_sum_set(f, a, b), l)
        }
        upper_sum_set_inf_exists_of_bounded(f, a, b, lb, ub)
        let u: Real satisfy {
            is_set_infimum(upper_sum_set(f, a, b), u)
        }
        forall(x: Real) {
            if lower_sum_set(f, a, b).contains(x) {
                lower_sum_set(f, a, b).contains(x) = lower_sum_contains(f, a, b, x)
                lower_sum_contains(f, a, b, x)
                let (p: Nat -> Real, n: Nat) satisfy {
                    is_partition(p, a, b, n) and x = lower_sum(f, p, n)
                }
                interior_constant_lower_sum_le_c_width(f, p, a, b, n, c, lb, ub)
                lower_sum(f, p, n) <= c * (b - a)
                x = lower_sum(f, p, n)
                x <= c * (b - a)
            }
        }
        is_set_upper_bound(lower_sum_set(f, a, b), c * (b - a))
        set_supremum_le_upper_bound(lower_sum_set(f, a, b), l, c * (b - a))
        l <= c * (b - a)
        forall(x: Real) {
            if upper_sum_set(f, a, b).contains(x) {
                upper_sum_set(f, a, b).contains(x) = upper_sum_contains(f, a, b, x)
                upper_sum_contains(f, a, b, x)
                let (p: Nat -> Real, n: Nat) satisfy {
                    is_partition(p, a, b, n) and x = upper_sum(f, p, n)
                }
                interior_constant_c_width_le_upper_sum(f, p, a, b, n, c, lb, ub)
                c * (b - a) <= upper_sum(f, p, n)
                x = upper_sum(f, p, n)
                c * (b - a) <= x
            }
        }
        is_set_lower_bound(upper_sum_set(f, a, b), c * (b - a))
        set_lower_bound_le_infimum(upper_sum_set(f, a, b), u, c * (b - a))
        c * (b - a) <= u
        lte_trans[Real](l, c * (b - a), u)
        l <= u
        interior_constant_darboux_condition(f, a, b, c, lb, ub)
        darboux_condition(f, a, b)
        darboux_eps_imp_inf_le_sup(f, a, b, l, u)
        u <= l
        lte_antisymm[Real](l, u)
        l = u
        c * (b - a) <= l
        l <= c * (b - a)
        lte_antisymm[Real](l, c * (b - a))
        l = c * (b - a)
        is_set_supremum(lower_sum_set(f, a, b), l) and is_set_infimum(upper_sum_set(f, a, b), l)
        exists(m: Real) {
            is_set_supremum(lower_sum_set(f, a, b), m) and is_set_infimum(upper_sum_set(f, a, b), m)
        }
        is_integrable(f, a, b)
    }
}

/// The integral over [a, b] of a bounded function constant on the open
/// interval (a, b) is c * (b - a).
theorem interior_constant_integral(f: Real -> Real, a: Real, b: Real, c: Real, lb: Real, ub: Real) {
    a <= b and
    (forall(t: Real) { a < t and t < b implies f(t) = c }) and
    (forall(t: Real) { interval_contains(a, b, t) implies lb <= f(t) }) and
    (forall(t: Real) { interval_contains(a, b, t) implies f(t) <= ub })
    implies
    integral(f, a, b) = c * (b - a)
} by {
    if a <= b and
       (forall(t: Real) { a < t and t < b implies f(t) = c }) and
       (forall(t: Real) { interval_contains(a, b, t) implies lb <= f(t) }) and
       (forall(t: Real) { interval_contains(a, b, t) implies f(t) <= ub }) {
        interior_constant_integrable(f, a, b, c, lb, ub)
        is_integrable(f, a, b)
        integral_spec(f, a, b)
        is_set_supremum(lower_sum_set(f, a, b), integral(f, a, b)) and is_set_infimum(upper_sum_set(f, a, b), integral(f, a, b))
        is_set_supremum(lower_sum_set(f, a, b), integral(f, a, b))
        forall(x: Real) {
            if lower_sum_set(f, a, b).contains(x) {
                lower_sum_set(f, a, b).contains(x) = lower_sum_contains(f, a, b, x)
                lower_sum_contains(f, a, b, x)
                let (p: Nat -> Real, n: Nat) satisfy {
                    is_partition(p, a, b, n) and x = lower_sum(f, p, n)
                }
                interior_constant_lower_sum_le_c_width(f, p, a, b, n, c, lb, ub)
                lower_sum(f, p, n) <= c * (b - a)
                x = lower_sum(f, p, n)
                x <= c * (b - a)
            }
        }
        sup_le_of_upper_bound(lower_sum_set(f, a, b), integral(f, a, b), c * (b - a))
        integral(f, a, b) <= c * (b - a)
        upper_sum_set_inf_exists_of_bounded(f, a, b, lb, ub)
        let u: Real satisfy {
            is_set_infimum(upper_sum_set(f, a, b), u)
        }
        forall(x: Real) {
            if upper_sum_set(f, a, b).contains(x) {
                upper_sum_set(f, a, b).contains(x) = upper_sum_contains(f, a, b, x)
                upper_sum_contains(f, a, b, x)
                let (p: Nat -> Real, n: Nat) satisfy {
                    is_partition(p, a, b, n) and x = upper_sum(f, p, n)
                }
                interior_constant_c_width_le_upper_sum(f, p, a, b, n, c, lb, ub)
                c * (b - a) <= upper_sum(f, p, n)
                x = upper_sum(f, p, n)
                c * (b - a) <= x
            }
        }
        is_set_lower_bound(upper_sum_set(f, a, b), c * (b - a))
        set_lower_bound_le_infimum(upper_sum_set(f, a, b), u, c * (b - a))
        c * (b - a) <= u
        is_set_infimum(upper_sum_set(f, a, b), integral(f, a, b)) and is_set_infimum(upper_sum_set(f, a, b), u)
        set_infimum_unique(upper_sum_set(f, a, b), integral(f, a, b), u)
        integral(f, a, b) = u
        c * (b - a) <= integral(f, a, b)
        lte_antisymm[Real](integral(f, a, b), c * (b - a))
        integral(f, a, b) = c * (b - a)
    }
}
