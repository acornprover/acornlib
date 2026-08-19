/// Mixed exponential and logarithm inequalities and classical constants.
/// /// This file collects inequalities relating `Real.exp` and `log` to each other and
/// to rational bounds, and establishes classical numerical facts such as
/// `2 < e < 3`, `e^2 > 4`, `1/2 < log 2 < 1` and `log 3` between one and two.

from nat import Nat, from_nat, pow_one, one_pow
from rat import Rat
from order import lt_trans, lt_of_lte_of_lt, not_gt_imp_lte
from real.exp import Real, exp_add, exp_zero, exp_pos, exp_increasing, exp_term, two, two_positive, three, e_gt_two, e_less_than_three, from_nat_two_eq, one_div_two_eq_half, pow_suc, factorial_pos, exp_ge_one_plus_x_plus_x2_half
from real.exp_deep import exp_two_e, exp_ge_partial, exp_partial_four, e_pow_two
from real.exp_inequalities import exp_monotone, exp_ge_one_plus_self
from real.exp_log_properties import log_strictly_increasing
from real.log import log_some_of_pos_exists, exp_log_or_zero, exp_neg
from real.log_exp_foundations import log_value_e
from real.log_deep import log_value_e_pow
from real.rpow_deep import rpow_gt_one_strict
from real.real_ring import mul_pos_pos, lt_mul_pos_left, mul_le_mul_nonneg, lte_mul_nonneg_right
from real.real_base import lt_add_pos, lte_lt_trans, lt_lte_trans, lt_add_right, lte_add_right, one_half_plus_one_half, gt_zero_imp_pos
from real.harmonic import real_one_div_pos, real_recip_antitone_pos
from list import partial

numerals Real

// =====================================================================
// Exponential upper bounds
// =====================================================================
// ===================================================================== 
/// For nonnegative `x`, the reciprocal of `1 + x` bounds `(-x).exp` from above.
theorem exp_upper_recip_one_plus_x(x: Real) {
    x >= Real.0 implies (-x).exp <= Real.1 / (Real.1 + x)
} by {
    if x >= Real.0 {
        exp_neg(x)
        (-x).exp = Real.1 / x.exp
        exp_ge_one_plus_self(x)
        x.exp >= Real.1 + x
        Real.1 + x <= x.exp
        Real.1 > Real.0
        Real.0 < Real.1
        lt_add_right(Real.0, Real.1, x)
        Real.0 + x < Real.1 + x
        Real.0 + x = x
        x < Real.1 + x
        Real.0 <= x
        lt_of_lte_of_lt(Real.0, x, Real.1 + x)
        Real.0 < Real.1 + x
        Real.1 + x > Real.0
        real_recip_antitone_pos(Real.1 + x, x.exp)
        Real.1 / x.exp <= Real.1 / (Real.1 + x)
        Real.1 / x.exp = (-x).exp
        (-x).exp <= Real.1 / (Real.1 + x)
    }
}

// =====================================================================
// Bounds on Euler's number
// =====================================================================
// ===================================================================== 
/// The square of Euler's number exceeds four.
theorem e_sq_gt_four {
    Real.e * Real.e > two * two
} by {
    exp_two_e
    two.exp = Real.e * Real.e
    two_positive
    two > Real.0
    two >= Real.0
    exp_ge_one_plus_x_plus_x2_half(two)
    two.exp >= Real.1 + two + two.pow(Nat.2) / two
    pow_suc(two, Nat.1)
    two.pow(Nat.1.suc) = two * two.pow(Nat.1)
    Nat.1.suc = Nat.2
    two.pow(Nat.2) = two * two.pow(Nat.1)
    pow_one[Real](two)
    two.pow(Nat.1) = two
    two.pow(Nat.2) = two * two
    two != Real.0
    two.pow(Nat.2) / two = two
    Real.1 + two + two.pow(Nat.2) / two = Real.1 + two + two
    Real.1 + two + two = Real.1 + two * two
    lt_add_pos(two * two, Real.1)
    Real.1.is_positive
    two * two < two * two + Real.1
    two * two + Real.1 = Real.1 + two * two
    two * two < Real.1 + two * two
    Real.1 + two * two = Real.1 + two + two.pow(Nat.2) / two
    two * two < Real.1 + two + two.pow(Nat.2) / two
    lt_lte_trans(two * two, Real.1 + two + two.pow(Nat.2) / two, two.exp)
    two * two < two.exp
    Real.e * Real.e > two * two
}

/// The square of Euler's number is below nine.
theorem e_sq_lt_nine {
    Real.e * Real.e < three * three
} by {
    e_less_than_three
    Real.e < three
    exp_pos(Real.1)
    (Real.1).exp > Real.0
    Real.e = (Real.1).exp
    Real.e > Real.0
    gt_zero_imp_pos(Real.e)
    Real.e.is_positive
    lt_mul_pos_left(Real.e, three, Real.e)
    Real.e * Real.e < Real.e * three
    three > Real.0
    gt_zero_imp_pos(three)
    three.is_positive
    lt_mul_pos_left(Real.e, three, three)
    Real.e * three < three * three
    lt_trans(Real.e * Real.e, Real.e * three, three * three)
    Real.e * Real.e < three * three
}

/// The exponential of two exceeds three.
theorem exp_two_gt_three {
    two.exp > three
} by {
    exp_two_e
    two.exp = Real.e * Real.e
    e_sq_gt_four
    Real.e * Real.e > two * two
    lt_add_pos(three, Real.1)
    Real.1.is_positive
    three < three + Real.1
    three + Real.1 = two * two
    three < two * two
    lt_trans(three, two * two, Real.e * Real.e)
    three < Real.e * Real.e
    two.exp > three
}

/// The exponential of two exceeds Euler's number.
theorem exp_two_gt_e {
    two.exp > Real.e
} by {
    exp_two_e
    two.exp = Real.e * Real.e
    e_gt_two
    Real.e > two
    Real.1 < two
    lt_trans(Real.1, two, Real.e)
    Real.1 < Real.e
    Real.e > Real.1
    gt_zero_imp_pos(Real.e)
    Real.e.is_positive
    lt_mul_pos_left(Real.1, Real.e, Real.e)
    Real.e * Real.1 < Real.e * Real.e
    Real.e * Real.1 = Real.e
    Real.e < Real.e * Real.e
    Real.e * Real.e > Real.e
    two.exp > Real.e
}

/// Euler's number exceeds five over two.
theorem e_gt_five_over_two {
    Real.e > Real.1 + Real.1 + Real.one_half
} by {
    exp_ge_partial(Real.1, Nat.4)
    partial(exp_term(Real.1), Nat.4) <= (Real.1).exp
    Real.e = (Real.1).exp
    partial(exp_term(Real.1), Nat.4) <= Real.e
    exp_partial_four(Real.1)
    partial(exp_term(Real.1), Nat.4) = Real.1 + Real.1 + Real.1.pow(Nat.2) / two + Real.1.pow(Nat.3) / Real.from_rat(Rat.from_nat(Nat.3.factorial))
    one_pow[Real](Nat.2)
    Real.1.pow(Nat.2) = Real.1
    one_pow[Real](Nat.3)
    Real.1.pow(Nat.3) = Real.1
    partial(exp_term(Real.1), Nat.4) = Real.1 + Real.1 + Real.1 / two + Real.1 / Real.from_rat(Rat.from_nat(Nat.3.factorial))
    one_div_two_eq_half
    Real.1 / two = Real.one_half
    partial(exp_term(Real.1), Nat.4) = Real.1 + Real.1 + Real.one_half + Real.1 / Real.from_rat(Rat.from_nat(Nat.3.factorial))
    factorial_pos(Nat.3)
    Real.from_rat(Rat.from_nat(Nat.3.factorial)) > Real.0
    real_one_div_pos(Real.from_rat(Rat.from_nat(Nat.3.factorial)))
    Real.1 / Real.from_rat(Rat.from_nat(Nat.3.factorial)) > Real.0
    gt_zero_imp_pos(Real.1 / Real.from_rat(Rat.from_nat(Nat.3.factorial)))
    (Real.1 / Real.from_rat(Rat.from_nat(Nat.3.factorial))).is_positive
    lt_add_pos(Real.1 + Real.1 + Real.one_half, Real.1 / Real.from_rat(Rat.from_nat(Nat.3.factorial)))
    Real.1 + Real.1 + Real.one_half < Real.1 + Real.1 + Real.one_half + Real.1 / Real.from_rat(Rat.from_nat(Nat.3.factorial))
    Real.1 + Real.1 + Real.one_half < partial(exp_term(Real.1), Nat.4)
    lt_lte_trans(Real.1 + Real.1 + Real.one_half, partial(exp_term(Real.1), Nat.4), Real.e)
    Real.1 + Real.1 + Real.one_half < Real.e
    Real.e > Real.1 + Real.1 + Real.one_half
}

// =====================================================================
// Classical logarithm values
// =====================================================================
// ===================================================================== 
/// Three is positive.
theorem three_pos {
    three > Real.0
} by {
    Real.0 < Real.1
    lt_add_right(Real.0, Real.1, Real.1)
    Real.0 + Real.1 < Real.1 + Real.1
    Real.0 + Real.1 = Real.1
    Real.1 < Real.1 + Real.1
    Real.1 + Real.1 = two
    Real.1 < two
    lt_add_pos(two, Real.1)
    Real.1.is_positive
    two < two + Real.1
    two + Real.1 = three
    lt_trans(Real.0, Real.1, two)
    Real.0 < two
    lt_trans(Real.0, two, three)
    Real.0 < three
    three > Real.0
}

/// The logarithm of three is below two.
theorem log_value_three_lt_two {
    three.log.get_or_else(Real.0) < two
} by {
    e_sq_gt_four
    Real.e * Real.e > two * two
    lt_add_pos(three, Real.1)
    Real.1.is_positive
    three < three + Real.1
    three + Real.1 = two * two
    three < two * two
    lt_trans(three, two * two, Real.e * Real.e)
    three < Real.e * Real.e
    three_pos
    three > Real.0
    exp_pos(Real.1)
    (Real.1).exp > Real.0
    Real.e = (Real.1).exp
    Real.e > Real.0
    gt_zero_imp_pos(Real.e)
    Real.e.is_positive
    mul_pos_pos(Real.e, Real.e)
    Real.e * Real.e > Real.0
    log_strictly_increasing(three, Real.e * Real.e)
    three.log.get_or_else(Real.0) < (Real.e * Real.e).log.get_or_else(Real.0)
    log_value_e_pow(Nat.2)
    (Real.e.pow(Nat.2)).log.get_or_else(Real.0) = from_nat[Real](Nat.2)
    Real.e.pow(Nat.2) = Real.e * Real.e
    from_nat_two_eq
    from_nat[Real](Nat.2) = two
    (Real.e * Real.e).log.get_or_else(Real.0) = two
    three.log.get_or_else(Real.0) < two
}

/// The logarithm of three exceeds one.
theorem log_value_three_gt_one {
    three.log.get_or_else(Real.0) > Real.1
} by {
    e_less_than_three
    Real.e < three
    exp_pos(Real.1)
    (Real.1).exp > Real.0
    Real.e = (Real.1).exp
    Real.e > Real.0
    three_pos
    three > Real.0
    log_strictly_increasing(Real.e, three)
    (Real.e).log.get_or_else(Real.0) < three.log.get_or_else(Real.0)
    log_value_e
    (Real.e).log.get_or_else(Real.0) = Real.1
    Real.1 < three.log.get_or_else(Real.0)
    three.log.get_or_else(Real.0) > Real.1
}

/// The logarithm of the square of Euler's number is two.
theorem log_value_e_sq {
    (Real.e * Real.e).log.get_or_else(Real.0) = two
} by {
    Real.e.pow(Nat.2) = Real.e * Real.e
    log_value_e_pow(Nat.2)
    (Real.e.pow(Nat.2)).log.get_or_else(Real.0) = from_nat[Real](Nat.2)
    (Real.e * Real.e).log.get_or_else(Real.0) = from_nat[Real](Nat.2)
    from_nat_two_eq
    from_nat[Real](Nat.2) = two
    (Real.e * Real.e).log.get_or_else(Real.0) = two
}

/// The logarithm of two exceeds one half.
theorem log_value_two_gt_half {
    two.log.get_or_else(Real.0) > Real.one_half
} by {
    if not two.log.get_or_else(Real.0) > Real.one_half {
        not_gt_imp_lte(two.log.get_or_else(Real.0), Real.one_half)
        two.log.get_or_else(Real.0) <= Real.one_half
        exp_monotone(two.log.get_or_else(Real.0), Real.one_half)
        (two.log.get_or_else(Real.0)).exp <= (Real.one_half).exp
        two_positive
        two > Real.0
        log_some_of_pos_exists(two)
        exists(y: Real) { two.log = Option.some(y) }
        let y: Real satisfy {
            two.log = Option.some(y)
        }
        exp_log_or_zero(two, y)
        y.exp = two
        two.log.get_or_else(Real.0) = y
        (two.log.get_or_else(Real.0)).exp = two
        two <= (Real.one_half).exp
        two_positive
        two > Real.0
        two >= Real.0
        exp_pos(Real.one_half)
        (Real.one_half).exp > Real.0
        (Real.one_half).exp >= Real.0
        two <= (Real.one_half).exp
        mul_le_mul_nonneg(two, two, (Real.one_half).exp, (Real.one_half).exp)
        two * two <= (Real.one_half).exp * (Real.one_half).exp
        exp_add(Real.one_half, Real.one_half)
        (Real.one_half + Real.one_half).exp = (Real.one_half).exp * (Real.one_half).exp
        one_half_plus_one_half
        Real.one_half + Real.one_half = Real.1
        (Real.1).exp = (Real.one_half).exp * (Real.one_half).exp
        Real.e = (Real.1).exp
        (Real.one_half).exp * (Real.one_half).exp = Real.e
        two * two <= Real.e
        e_less_than_three
        Real.e < three
        lt_add_pos(three, Real.1)
        Real.1.is_positive
        three < three + Real.1
        three + Real.1 = two * two
        three < two * two
        lt_trans(Real.e, three, two * two)
        Real.e < two * two
        lt_of_lte_of_lt(two * two, Real.e, two * two)
        two * two < two * two
        false
    }
    two.log.get_or_else(Real.0) > Real.one_half
}

/// A positive real power of Euler's number exceeds one.
theorem rpow_e_gt_one_strict(a: Real, u: Real) {
    a > Real.0 and (Real.e).rpow(a) = Option.some(u)
    implies u > Real.1
} by {
    if a > Real.0 and (Real.e).rpow(a) = Option.some(u) {
        e_gt_two
        Real.e > two
        Real.0 < Real.1
        lt_add_right(Real.0, Real.1, Real.1)
        Real.0 + Real.1 < Real.1 + Real.1
        Real.0 + Real.1 = Real.1
        Real.1 < Real.1 + Real.1
        Real.1 + Real.1 = two
        Real.1 < two
        lt_trans(Real.1, two, Real.e)
        Real.1 < Real.e
        Real.e > Real.1
        rpow_gt_one_strict(Real.e, a, u)
        u > Real.1
    }
}
