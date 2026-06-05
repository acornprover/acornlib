from real.continuity_algebra import add_fns_continuous, add_fns_continuous_at
from real.continuity_composition import constant_function_is_continuous,
    constant_function_is_continuous_at
from real.continuity_base import Real, add_fns, continuous, continuous_at

/// The function obtained by adding the fixed real constant c to f on the left.
define const_add_left(c: Real, f: Real -> Real, x: Real) -> Real {
    c + f(x)
}

/// The function obtained by adding the fixed real constant c to f on the right.
define const_add_right(f: Real -> Real, c: Real, x: Real) -> Real {
    f(x) + c
}

/// Adding a constant on the left agrees with add_fns of the constant function and f.
theorem add_fns_constant_left_eq(c: Real, f: Real -> Real) {
    add_fns[Real](constant[Real, Real](c), f) = const_add_left(c, f)
} by {
    forall(x: Real) {
        constant[Real, Real](c, x) = c
        add_fns[Real](constant[Real, Real](c), f, x) = c + f(x)
        const_add_left(c, f, x) = c + f(x)
        add_fns[Real](constant[Real, Real](c), f, x) = const_add_left(c, f, x)
    }
}

/// Adding a constant on the right agrees with add_fns of f and the constant function.
theorem add_fns_constant_right_eq(f: Real -> Real, c: Real) {
    add_fns[Real](f, constant[Real, Real](c)) = const_add_right(f, c)
} by {
    forall(x: Real) {
        constant[Real, Real](c, x) = c
        add_fns[Real](f, constant[Real, Real](c), x) = f(x) + c
        const_add_right(f, c, x) = f(x) + c
        add_fns[Real](f, constant[Real, Real](c), x) = const_add_right(f, c, x)
    }
}

/// Adding a fixed real constant on the left preserves continuity at a point.
theorem continuous_at_const_add_left(c: Real, f: Real -> Real, x: Real) {
    continuous_at(f, x)
    implies continuous_at(const_add_left(c, f), x)
} by {
    constant_function_is_continuous_at(c, x)
    continuous_at(constant[Real, Real](c), x)
    add_fns_continuous_at(constant[Real, Real](c), f, x)
    continuous_at(add_fns[Real](constant[Real, Real](c), f), x)
    add_fns_constant_left_eq(c, f)
    continuous_at(const_add_left(c, f), x)
}

/// Adding a fixed real constant on the left preserves continuous functions.
theorem continuous_const_add_left(c: Real, f: Real -> Real) {
    continuous(f) implies continuous(const_add_left(c, f))
} by {
    constant_function_is_continuous(c)
    continuous(constant[Real, Real](c))
    add_fns_continuous(constant[Real, Real](c), f)
    continuous(add_fns[Real](constant[Real, Real](c), f))
    add_fns_constant_left_eq(c, f)
    continuous(const_add_left(c, f))
}

/// Adding a fixed real constant on the right preserves continuity at a point.
theorem continuous_at_const_add_right(f: Real -> Real, c: Real, x: Real) {
    continuous_at(f, x)
    implies continuous_at(const_add_right(f, c), x)
} by {
    constant_function_is_continuous_at(c, x)
    continuous_at(constant[Real, Real](c), x)
    add_fns_continuous_at(f, constant[Real, Real](c), x)
    continuous_at(add_fns[Real](f, constant[Real, Real](c)), x)
    add_fns_constant_right_eq(f, c)
    continuous_at(const_add_right(f, c), x)
}

/// Adding a fixed real constant on the right preserves continuous functions.
theorem continuous_const_add_right(f: Real -> Real, c: Real) {
    continuous(f) implies continuous(const_add_right(f, c))
} by {
    constant_function_is_continuous(c)
    continuous(constant[Real, Real](c))
    add_fns_continuous(f, constant[Real, Real](c))
    continuous(add_fns[Real](f, constant[Real, Real](c)))
    add_fns_constant_right_eq(f, c)
    continuous(const_add_right(f, c))
}
