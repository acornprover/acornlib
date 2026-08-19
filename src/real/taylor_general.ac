/// Taylor's theorem with Lagrange remainder, for the general n-th order
/// Taylor polynomial.
///
/// The mean value theorem says that a differentiable function's chord slope is
/// attained as a derivative value; Taylor's theorem is its n-th order
/// generalization: the value of an (n + 1)-times differentiable function at b
/// is its n-th order Taylor polynomial at a plus a Lagrange remainder
/// f^(n+1)(c) (b - a)^(n + 1) / (n + 1)! for some interior point c.
///
/// The derivative is expressed in this library as a predicate
/// (has_derivative_at) rather than as a function, so there is no computable
/// "derivative of derivative".  Instead the n-th iterated derivative is
/// represented by a chain dfs of derivative functions: dfs(0) is f itself and
/// dfs(k + 1) is a derivative function of dfs(k).  The value of the k-th
/// iterated derivative at a point x is iterated_derivative(f, dfs, k, x), which
/// is dfs(k, x).
///
/// This file states the general theorem and proves its instances at orders
/// zero, one, two and three:
///   - the order-zero instance is exactly the mean value theorem,
///   - the order-one instance is the second-order Taylor theorem of taylor.ac,
///   - the order-two and order-three instances (third- and fourth-derivative
///     Lagrange remainders) are proved directly by the classical single
///     application of Rolle's theorem to a carefully chosen auxiliary function.
/// A proof of the general statement by induction would require a telescoping
/// identity for the derivative of the auxiliary function over a variable
/// summation range, which is not yet present in the library.

from order import lt_trans, lt_imp_ne_symm, lt_imp_ne
from order_set import closed_interval_set
from nat import Nat, from_nat, from_nat_zero, from_nat_one, from_nat_add, from_nat_mul, pow_zero, pow_one, one_pow, pow_add, factorial_zero, factorial_one, factorial_step, alt_induction
from rat import Rat
from list import partial, partial_split_last, partial_shift_suc, partial_zero, partial_one, partial_pointwise_eq
from real.continuity_base import Real, continuous, continuous_at
from real.derivative_basic import has_derivative_at, has_derivative_at_unique, sub_ne_zero_of_ne, constant_has_derivative_at
from real.derivative_rules import derivative_pointwise_sub, derivative_pointwise_add, derivative_pointwise_neg
from real.derivative_linear import derivative_pointwise_const_mul
from real.derivative_product import derivative_pointwise_mul, derivative_pointwise_square
from real.derivative_polynomial_affine import derivative_square_real_after_affine, derivative_cube_real_after_affine
from real.derivative_affine_named import affine_real_has_derivative_at, affine_real_eq_pointwise_affine
from real.derivative_continuity import div_mul_cancel_denominator
from real.calculus_api import is_derivative_fn, is_derivative_fn_at, is_derivative_fn_iff, is_derivative_fn_imp_continuous_at
from real.continuity_affine import affine_real, continuous_at_affine_real
from real.continuity_square import square_real, continuous_at_square_real
from real.continuity_cube import cube_real, continuous_at_cube_real
from real.continuity_pointwise import continuous_at_pointwise_neg, continuous_at_pointwise_add
from real.continuity_pointwise_mul import continuous_at_pointwise_mul
from real.continuity_composition import continuous_at_compose, constant_function_is_continuous_at
from real.real_base import add_comm, add_assoc, add_zero_right, add_zero_left, add_neg_eq_zero, neg_zero, neg_distrib, neg_neg
from real.real_ring import mul_zero_left, mul_zero_right, real_mul_comm, mul_assoc, mul_one_right, mul_one_left, mul_distrib_right, mul_distrib_left, mul_neg_right, mul_neg_left, from_nat_is_from_rat
from real.real_field import mul_inverse, mul_left_cancel
from real.real_seq import sub_zero_imp_eq
from real.mean_value import continuous_on_closed, differentiable_on_open, is_derivative_on_open, is_derivative_on_open_imp_differentiable_on_open, rolle_theorem, mean_value_theorem, secant_slope
from real.taylor import taylor_two, taylor_two_not_zero, taylor_order_one, taylor_order_two, taylor2_sub_add_cancel, taylor2_sub_sub_add, is_derivative_fn_imp_is_derivative_on_open
from real.exp import factorial_pos, factorial_suc_real, pow_suc, suc_pos
from algebra.field.field import mul_not_zero, field_square_nonzero, inverse_not_zero, inverse_dist, inverse_one
from data.basic.function_algebra import pointwise_add, pointwise_neg, pointwise_mul
from data.basic.functions import identity_fn, compose, function_extensionality, function_eq_transport_predicate_rev

numerals Real

/// The real number three, written as 1 + 1 + 1.
let taylor_three: Real = Real.1 + Real.1 + Real.1

/// The real number six, written as 2 * 3.
let taylor_six: Real = taylor_two * taylor_three

/// The image of two in the reals is two.
theorem from_nat_two_real {
    from_nat[Real](Nat.2) = taylor_two
} by {
    from_nat_add[Real](Nat.1, Nat.1)
    from_nat[Real](Nat.1 + Nat.1) = from_nat[Real](Nat.1) + from_nat[Real](Nat.1)
    from_nat_one[Real]
    from_nat[Real](Nat.1) = Real.1
    from_nat[Real](Nat.1 + Nat.1) = Real.1 + Real.1
    Nat.1 + Nat.1 = Nat.2
    from_nat[Real](Nat.2) = Real.1 + Real.1
    taylor_two = Real.1 + Real.1
    from_nat[Real](Nat.2) = taylor_two
}

/// The image of three in the reals is three.
theorem from_nat_three_real {
    from_nat[Real](Nat.3) = taylor_three
} by {
    from_nat_add[Real](Nat.2, Nat.1)
    from_nat[Real](Nat.2 + Nat.1) = from_nat[Real](Nat.2) + from_nat[Real](Nat.1)
    from_nat_two_real
    from_nat[Real](Nat.2) = taylor_two
    from_nat_one[Real]
    from_nat[Real](Nat.1) = Real.1
    from_nat[Real](Nat.2 + Nat.1) = taylor_two + Real.1
    Nat.2 + Nat.1 = Nat.3
    from_nat[Real](Nat.3) = taylor_two + Real.1
    taylor_three = Real.1 + Real.1 + Real.1
    taylor_two = Real.1 + Real.1
    taylor_two + Real.1 = Real.1 + Real.1 + Real.1
    taylor_two + Real.1 = taylor_three
    from_nat[Real](Nat.3) = taylor_three
}

/// The image of six in the reals is six.
theorem from_nat_six_real {
    from_nat[Real](Nat.6) = taylor_six
} by {
    from_nat_mul[Real](Nat.3, Nat.2)
    from_nat[Real](Nat.3 * Nat.2) = from_nat[Real](Nat.3) * from_nat[Real](Nat.2)
    from_nat_three_real
    from_nat[Real](Nat.3) = taylor_three
    from_nat_two_real
    from_nat[Real](Nat.2) = taylor_two
    from_nat[Real](Nat.3 * Nat.2) = taylor_three * taylor_two
    Nat.3 * Nat.2 = Nat.6
    from_nat[Real](Nat.6) = taylor_three * taylor_two
    taylor_six = taylor_two * taylor_three
    real_mul_comm(taylor_two, taylor_three)
    taylor_two * taylor_three = taylor_three * taylor_two
    from_nat[Real](Nat.6) = taylor_six
}

/// The real number three is nonzero.
theorem taylor_three_not_zero {
    taylor_three != Real.0
} by {
    from_nat_three_real
    from_nat[Real](Nat.3) = taylor_three
    taylor_three = from_nat[Real](Nat.3)
    suc_pos(Nat.2)
    Real.from_rat(Rat.from_nat(Nat.2.suc)) > Real.0
    Nat.2.suc = Nat.3
    Real.from_rat(Rat.from_nat(Nat.3)) > Real.0
    from_nat_is_from_rat(Nat.3)
    from_nat[Real](Nat.3) = Real.from_rat(Rat.from_nat(Nat.3))
    from_nat[Real](Nat.3) > Real.0
    lt_imp_ne_symm(Real.0, from_nat[Real](Nat.3))
    from_nat[Real](Nat.3) != Real.0
    taylor_three != Real.0
}

/// The real number six is nonzero.
theorem taylor_six_not_zero {
    taylor_six != Real.0
} by {
    taylor_six = taylor_two * taylor_three
    taylor_two_not_zero
    taylor_three_not_zero
    mul_not_zero[Real](taylor_two, taylor_three)
    taylor_two * taylor_three != Real.0
    taylor_six != Real.0
}

/// The k-th iterated derivative of f along a chain dfs of derivative functions,
/// evaluated at x: dfs(0) is f itself and dfs(k + 1) is a derivative function
/// of dfs(k), so iterated_derivative(f, dfs, k, x) is the value of the k-th
/// derivative of f at x.
define iterated_derivative(f: Real -> Real, dfs: Nat -> Real -> Real, k: Nat, x: Real) -> Real {
    dfs(k, x)
}

/// True if dfs is a chain of derivative functions for f of length n: dfs(0) is
/// f and dfs(k + 1) is a derivative function of dfs(k) for every k < n.
define is_derivative_chain(f: Real -> Real, dfs: Nat -> Real -> Real, n: Nat) -> Bool {
    dfs(Nat.0) = f and forall(k: Nat) {
        k < n implies is_derivative_fn(dfs(k), dfs(k.suc))
    }
}

/// The k-th term of the Taylor polynomial of f at a with derivative chain dfs:
/// the value of the k-th derivative at a times (x - a)^k divided by k!.
define taylor_term(f: Real -> Real, dfs: Nat -> Real -> Real, a: Real, x: Real, k: Nat) -> Real {
    (iterated_derivative(f, dfs, k, a) / from_nat[Real](k.factorial)) * (x - a).pow(k)
}

/// The n-th order Taylor polynomial of f at a, evaluated at x: the sum of the
/// Taylor terms for k = 0, ..., n.
define taylor_poly(f: Real -> Real, dfs: Nat -> Real -> Real, a: Real, x: Real, n: Nat) -> Real {
    partial(taylor_term(f, dfs, a, x), n.suc)
}

/// The Lagrange remainder term of order n: the value of the (n + 1)-st
/// derivative at c times (x - a)^(n + 1) divided by (n + 1)!.
define taylor_remainder(f: Real -> Real, dfs: Nat -> Real -> Real, a: Real, x: Real, c: Real, n: Nat) -> Real {
    (iterated_derivative(f, dfs, n.suc, c) / from_nat[Real](n.suc.factorial)) * (x - a).pow(n.suc)
}

/// The derivative chain of length one for f: dfs(0) = f and dfs(k) = df for k >= 1.
define taylor_chain1(f: Real -> Real, df: Real -> Real, k: Nat) -> Real -> Real {
    match k {
        Nat.zero { f }
        Nat.suc(m) { df }
    }
}

/// The derivative chain of length two for f: dfs(0) = f, dfs(1) = df and
/// dfs(k) = ddf for k >= 2.
define taylor_chain2(f: Real -> Real, df: Real -> Real, ddf: Real -> Real, k: Nat) -> Real -> Real {
    match k {
        Nat.zero { f }
        Nat.suc(m) {
            match m {
                Nat.zero { df }
                Nat.suc(m2) { ddf }
            }
        }
    }
}

/// The second tail of the chain of length three, starting at index two.
define taylor_chain3_tail2(ddf: Real -> Real, dddf: Real -> Real, k: Nat) -> Real -> Real {
    match k {
        Nat.zero { ddf }
        Nat.suc(m) { dddf }
    }
}

/// The tail of the chain of length three, starting at index one.
define taylor_chain3_tail(df: Real -> Real, ddf: Real -> Real, dddf: Real -> Real, k: Nat) -> Real -> Real {
    match k {
        Nat.zero { df }
        Nat.suc(m) { taylor_chain3_tail2(ddf, dddf, m) }
    }
}

/// The derivative chain of length three for f: dfs(0) = f, dfs(1) = df,
/// dfs(2) = ddf and dfs(k) = dddf for k >= 3.
define taylor_chain3(f: Real -> Real, df: Real -> Real, ddf: Real -> Real, dddf: Real -> Real, k: Nat) -> Real -> Real {
    match k {
        Nat.zero { f }
        Nat.suc(m) { taylor_chain3_tail(df, ddf, dddf, m) }
    }
}

/// The third tail of the chain of length four, starting at index three.
define taylor_chain4_tail3(dddf: Real -> Real, dddd: Real -> Real, k: Nat) -> Real -> Real {
    match k {
        Nat.zero { dddf }
        Nat.suc(m) { dddd }
    }
}

/// The second tail of the chain of length four, starting at index two.
define taylor_chain4_tail2(ddf: Real -> Real, dddf: Real -> Real, dddd: Real -> Real, k: Nat) -> Real -> Real {
    match k {
        Nat.zero { ddf }
        Nat.suc(m) { taylor_chain4_tail3(dddf, dddd, m) }
    }
}

/// The tail of the chain of length four, starting at index one.
define taylor_chain4_tail(df: Real -> Real, ddf: Real -> Real, dddf: Real -> Real, dddd: Real -> Real, k: Nat) -> Real -> Real {
    match k {
        Nat.zero { df }
        Nat.suc(m) { taylor_chain4_tail2(ddf, dddf, dddd, m) }
    }
}

/// The derivative chain of length four for f: dfs(0) = f, dfs(1) = df,
/// dfs(2) = ddf, dfs(3) = dddf and dfs(k) = dddd for k >= 4.
define taylor_chain4(f: Real -> Real, df: Real -> Real, ddf: Real -> Real, dddf: Real -> Real, dddd: Real -> Real, k: Nat) -> Real -> Real {
    match k {
        Nat.zero { f }
        Nat.suc(m) { taylor_chain4_tail(df, ddf, dddf, dddd, m) }
    }
}

/// The first element of the chain of length one is f.
theorem taylor_chain1_zero(f: Real -> Real, df: Real -> Real) {
    taylor_chain1(f, df, Nat.0) = f
}

/// The remaining elements of the chain of length one are df.
theorem taylor_chain1_suc(f: Real -> Real, df: Real -> Real, k: Nat) {
    taylor_chain1(f, df, k.suc) = df
}

/// The first element of the chain of length two is f.
theorem taylor_chain2_zero(f: Real -> Real, df: Real -> Real, ddf: Real -> Real) {
    taylor_chain2(f, df, ddf, Nat.0) = f
}

/// The second element of the chain of length two is df.
theorem taylor_chain2_one(f: Real -> Real, df: Real -> Real, ddf: Real -> Real) {
    taylor_chain2(f, df, ddf, Nat.1) = df
}

/// The remaining elements of the chain of length two are ddf.
theorem taylor_chain2_suc_suc(f: Real -> Real, df: Real -> Real, ddf: Real -> Real, k: Nat) {
    taylor_chain2(f, df, ddf, k.suc.suc) = ddf
}

/// The first element of the chain of length three is f.
theorem taylor_chain3_zero(f: Real -> Real, df: Real -> Real, ddf: Real -> Real, dddf: Real -> Real) {
    taylor_chain3(f, df, ddf, dddf, Nat.0) = f
}

/// The successor of a chain of length three is its tail.
theorem taylor_chain3_suc(f: Real -> Real, df: Real -> Real, ddf: Real -> Real, dddf: Real -> Real, k: Nat) {
    taylor_chain3(f, df, ddf, dddf, k.suc) = taylor_chain3_tail(df, ddf, dddf, k)
}

/// The tail of a chain of length three at zero is df.
theorem taylor_chain3_tail_zero(df: Real -> Real, ddf: Real -> Real, dddf: Real -> Real) {
    taylor_chain3_tail(df, ddf, dddf, Nat.0) = df
}

/// The successor of a tail of length three is its second tail.
theorem taylor_chain3_tail_suc(df: Real -> Real, ddf: Real -> Real, dddf: Real -> Real, k: Nat) {
    taylor_chain3_tail(df, ddf, dddf, k.suc) = taylor_chain3_tail2(ddf, dddf, k)
}

/// The second tail of a chain of length three at zero is ddf.
theorem taylor_chain3_tail2_zero(ddf: Real -> Real, dddf: Real -> Real) {
    taylor_chain3_tail2(ddf, dddf, Nat.0) = ddf
}

/// The successor of the second tail of a chain of length three is dddf.
theorem taylor_chain3_tail2_suc(ddf: Real -> Real, dddf: Real -> Real, k: Nat) {
    taylor_chain3_tail2(ddf, dddf, k.suc) = dddf
}

/// The second element of the chain of length three is df.
theorem taylor_chain3_one(f: Real -> Real, df: Real -> Real, ddf: Real -> Real, dddf: Real -> Real) {
    taylor_chain3(f, df, ddf, dddf, Nat.1) = df
} by {
    taylor_chain3_suc(f, df, ddf, dddf, Nat.0)
    taylor_chain3(f, df, ddf, dddf, Nat.0.suc) = taylor_chain3_tail(df, ddf, dddf, Nat.0)
    Nat.0.suc = Nat.1
    taylor_chain3(f, df, ddf, dddf, Nat.1) = taylor_chain3_tail(df, ddf, dddf, Nat.0)
    taylor_chain3_tail_zero(df, ddf, dddf)
    taylor_chain3_tail(df, ddf, dddf, Nat.0) = df
    taylor_chain3(f, df, ddf, dddf, Nat.1) = df
}

/// The third element of the chain of length three is ddf.
theorem taylor_chain3_two(f: Real -> Real, df: Real -> Real, ddf: Real -> Real, dddf: Real -> Real) {
    taylor_chain3(f, df, ddf, dddf, Nat.2) = ddf
} by {
    taylor_chain3_suc(f, df, ddf, dddf, Nat.1)
    taylor_chain3(f, df, ddf, dddf, Nat.1.suc) = taylor_chain3_tail(df, ddf, dddf, Nat.1)
    Nat.1.suc = Nat.2
    taylor_chain3(f, df, ddf, dddf, Nat.2) = taylor_chain3_tail(df, ddf, dddf, Nat.1)
    taylor_chain3_tail_suc(df, ddf, dddf, Nat.0)
    taylor_chain3_tail(df, ddf, dddf, Nat.0.suc) = taylor_chain3_tail2(ddf, dddf, Nat.0)
    Nat.0.suc = Nat.1
    taylor_chain3_tail(df, ddf, dddf, Nat.1) = taylor_chain3_tail2(ddf, dddf, Nat.0)
    taylor_chain3_tail2_zero(ddf, dddf)
    taylor_chain3_tail2(ddf, dddf, Nat.0) = ddf
    taylor_chain3_tail(df, ddf, dddf, Nat.1) = ddf
    taylor_chain3(f, df, ddf, dddf, Nat.2) = ddf
}

/// The fourth element of the chain of length three is dddf.
theorem taylor_chain3_three(f: Real -> Real, df: Real -> Real, ddf: Real -> Real, dddf: Real -> Real) {
    taylor_chain3(f, df, ddf, dddf, Nat.3) = dddf
} by {
    taylor_chain3_suc(f, df, ddf, dddf, Nat.2)
    taylor_chain3(f, df, ddf, dddf, Nat.2.suc) = taylor_chain3_tail(df, ddf, dddf, Nat.2)
    Nat.2.suc = Nat.3
    taylor_chain3(f, df, ddf, dddf, Nat.3) = taylor_chain3_tail(df, ddf, dddf, Nat.2)
    taylor_chain3_tail_suc(df, ddf, dddf, Nat.1)
    taylor_chain3_tail(df, ddf, dddf, Nat.1.suc) = taylor_chain3_tail2(ddf, dddf, Nat.1)
    Nat.1.suc = Nat.2
    taylor_chain3_tail(df, ddf, dddf, Nat.2) = taylor_chain3_tail2(ddf, dddf, Nat.1)
    taylor_chain3_tail2_suc(ddf, dddf, Nat.0)
    taylor_chain3_tail2(ddf, dddf, Nat.0.suc) = dddf
    Nat.0.suc = Nat.1
    taylor_chain3_tail2(ddf, dddf, Nat.1) = dddf
    taylor_chain3_tail(df, ddf, dddf, Nat.2) = dddf
    taylor_chain3(f, df, ddf, dddf, Nat.3) = dddf
}

/// The first element of the chain of length four is f.
theorem taylor_chain4_zero(f: Real -> Real, df: Real -> Real, ddf: Real -> Real, dddf: Real -> Real, dddd: Real -> Real) {
    taylor_chain4(f, df, ddf, dddf, dddd, Nat.0) = f
}

/// The successor of a chain of length four is its tail.
theorem taylor_chain4_suc(f: Real -> Real, df: Real -> Real, ddf: Real -> Real, dddf: Real -> Real, dddd: Real -> Real, k: Nat) {
    taylor_chain4(f, df, ddf, dddf, dddd, k.suc) = taylor_chain4_tail(df, ddf, dddf, dddd, k)
}

/// The tail of a chain of length four at zero is df.
theorem taylor_chain4_tail_zero(df: Real -> Real, ddf: Real -> Real, dddf: Real -> Real, dddd: Real -> Real) {
    taylor_chain4_tail(df, ddf, dddf, dddd, Nat.0) = df
}

/// The successor of a tail of length four is its second tail.
theorem taylor_chain4_tail_suc(df: Real -> Real, ddf: Real -> Real, dddf: Real -> Real, dddd: Real -> Real, k: Nat) {
    taylor_chain4_tail(df, ddf, dddf, dddd, k.suc) = taylor_chain4_tail2(ddf, dddf, dddd, k)
}

/// The second tail of a chain of length four at zero is ddf.
theorem taylor_chain4_tail2_zero(ddf: Real -> Real, dddf: Real -> Real, dddd: Real -> Real) {
    taylor_chain4_tail2(ddf, dddf, dddd, Nat.0) = ddf
}

/// The successor of the second tail of a chain of length four is its third tail.
theorem taylor_chain4_tail2_suc(ddf: Real -> Real, dddf: Real -> Real, dddd: Real -> Real, k: Nat) {
    taylor_chain4_tail2(ddf, dddf, dddd, k.suc) = taylor_chain4_tail3(dddf, dddd, k)
}

/// The third tail of a chain of length four at zero is dddf.
theorem taylor_chain4_tail3_zero(dddf: Real -> Real, dddd: Real -> Real) {
    taylor_chain4_tail3(dddf, dddd, Nat.0) = dddf
}

/// The successor of the third tail of a chain of length four is dddd.
theorem taylor_chain4_tail3_suc(dddf: Real -> Real, dddd: Real -> Real, k: Nat) {
    taylor_chain4_tail3(dddf, dddd, k.suc) = dddd
}

/// The second element of the chain of length four is df.
theorem taylor_chain4_one(f: Real -> Real, df: Real -> Real, ddf: Real -> Real, dddf: Real -> Real, dddd: Real -> Real) {
    taylor_chain4(f, df, ddf, dddf, dddd, Nat.1) = df
} by {
    taylor_chain4_suc(f, df, ddf, dddf, dddd, Nat.0)
    taylor_chain4(f, df, ddf, dddf, dddd, Nat.0.suc) = taylor_chain4_tail(df, ddf, dddf, dddd, Nat.0)
    Nat.0.suc = Nat.1
    taylor_chain4(f, df, ddf, dddf, dddd, Nat.1) = taylor_chain4_tail(df, ddf, dddf, dddd, Nat.0)
    taylor_chain4_tail_zero(df, ddf, dddf, dddd)
    taylor_chain4_tail(df, ddf, dddf, dddd, Nat.0) = df
    taylor_chain4(f, df, ddf, dddf, dddd, Nat.1) = df
}

/// The third element of the chain of length four is ddf.
theorem taylor_chain4_two(f: Real -> Real, df: Real -> Real, ddf: Real -> Real, dddf: Real -> Real, dddd: Real -> Real) {
    taylor_chain4(f, df, ddf, dddf, dddd, Nat.2) = ddf
} by {
    taylor_chain4_suc(f, df, ddf, dddf, dddd, Nat.1)
    taylor_chain4(f, df, ddf, dddf, dddd, Nat.1.suc) = taylor_chain4_tail(df, ddf, dddf, dddd, Nat.1)
    Nat.1.suc = Nat.2
    taylor_chain4(f, df, ddf, dddf, dddd, Nat.2) = taylor_chain4_tail(df, ddf, dddf, dddd, Nat.1)
    taylor_chain4_tail_suc(df, ddf, dddf, dddd, Nat.0)
    taylor_chain4_tail(df, ddf, dddf, dddd, Nat.0.suc) = taylor_chain4_tail2(ddf, dddf, dddd, Nat.0)
    Nat.0.suc = Nat.1
    taylor_chain4_tail(df, ddf, dddf, dddd, Nat.1) = taylor_chain4_tail2(ddf, dddf, dddd, Nat.0)
    taylor_chain4_tail2_zero(ddf, dddf, dddd)
    taylor_chain4_tail2(ddf, dddf, dddd, Nat.0) = ddf
    taylor_chain4_tail(df, ddf, dddf, dddd, Nat.1) = ddf
    taylor_chain4(f, df, ddf, dddf, dddd, Nat.2) = ddf
}

/// The fourth element of the chain of length four is dddf.
theorem taylor_chain4_three(f: Real -> Real, df: Real -> Real, ddf: Real -> Real, dddf: Real -> Real, dddd: Real -> Real) {
    taylor_chain4(f, df, ddf, dddf, dddd, Nat.3) = dddf
} by {
    taylor_chain4_suc(f, df, ddf, dddf, dddd, Nat.2)
    taylor_chain4(f, df, ddf, dddf, dddd, Nat.2.suc) = taylor_chain4_tail(df, ddf, dddf, dddd, Nat.2)
    Nat.2.suc = Nat.3
    taylor_chain4(f, df, ddf, dddf, dddd, Nat.3) = taylor_chain4_tail(df, ddf, dddf, dddd, Nat.2)
    taylor_chain4_tail_suc(df, ddf, dddf, dddd, Nat.1)
    taylor_chain4_tail(df, ddf, dddf, dddd, Nat.1.suc) = taylor_chain4_tail2(ddf, dddf, dddd, Nat.1)
    Nat.1.suc = Nat.2
    taylor_chain4_tail(df, ddf, dddf, dddd, Nat.2) = taylor_chain4_tail2(ddf, dddf, dddd, Nat.1)
    taylor_chain4_tail2_suc(ddf, dddf, dddd, Nat.0)
    taylor_chain4_tail2(ddf, dddf, dddd, Nat.0.suc) = taylor_chain4_tail3(dddf, dddd, Nat.0)
    Nat.0.suc = Nat.1
    taylor_chain4_tail2(ddf, dddf, dddd, Nat.1) = taylor_chain4_tail3(dddf, dddd, Nat.0)
    taylor_chain4_tail3_zero(dddf, dddd)
    taylor_chain4_tail3(dddf, dddd, Nat.0) = dddf
    taylor_chain4_tail2(ddf, dddf, dddd, Nat.1) = dddf
    taylor_chain4_tail(df, ddf, dddf, dddd, Nat.2) = dddf
    taylor_chain4(f, df, ddf, dddf, dddd, Nat.3) = dddf
}

/// The fifth element of the chain of length four is dddd.
theorem taylor_chain4_four(f: Real -> Real, df: Real -> Real, ddf: Real -> Real, dddf: Real -> Real, dddd: Real -> Real) {
    taylor_chain4(f, df, ddf, dddf, dddd, Nat.4) = dddd
} by {
    taylor_chain4_suc(f, df, ddf, dddf, dddd, Nat.3)
    taylor_chain4(f, df, ddf, dddf, dddd, Nat.3.suc) = taylor_chain4_tail(df, ddf, dddf, dddd, Nat.3)
    Nat.3.suc = Nat.4
    taylor_chain4(f, df, ddf, dddf, dddd, Nat.4) = taylor_chain4_tail(df, ddf, dddf, dddd, Nat.3)
    taylor_chain4_tail_suc(df, ddf, dddf, dddd, Nat.2)
    taylor_chain4_tail(df, ddf, dddf, dddd, Nat.2.suc) = taylor_chain4_tail2(ddf, dddf, dddd, Nat.2)
    Nat.2.suc = Nat.3
    taylor_chain4_tail(df, ddf, dddf, dddd, Nat.3) = taylor_chain4_tail2(ddf, dddf, dddd, Nat.2)
    taylor_chain4_tail2_suc(ddf, dddf, dddd, Nat.1)
    taylor_chain4_tail2(ddf, dddf, dddd, Nat.1.suc) = taylor_chain4_tail3(dddf, dddd, Nat.1)
    Nat.1.suc = Nat.2
    taylor_chain4_tail2(ddf, dddf, dddd, Nat.2) = taylor_chain4_tail3(dddf, dddd, Nat.1)
    taylor_chain4_tail3_suc(dddf, dddd, Nat.0)
    taylor_chain4_tail3(dddf, dddd, Nat.0.suc) = dddd
    Nat.0.suc = Nat.1
    taylor_chain4_tail3(dddf, dddd, Nat.1) = dddd
    taylor_chain4_tail2(ddf, dddf, dddd, Nat.2) = dddd
    taylor_chain4_tail(df, ddf, dddf, dddd, Nat.3) = dddd
    taylor_chain4(f, df, ddf, dddf, dddd, Nat.4) = dddd
}

/// The factorial of two is two.
theorem factorial_two_nat {
    Nat.2.factorial = Nat.2
} by {
    factorial_step(Nat.1)
    Nat.1.suc.factorial = Nat.1.suc * Nat.1.factorial
    Nat.1.suc = Nat.2
    Nat.2.factorial = Nat.1.suc * Nat.1.factorial
    factorial_step(Nat.0)
    Nat.0.suc.factorial = Nat.0.suc * Nat.0.factorial
    Nat.0.suc = Nat.1
    Nat.1.factorial = Nat.1 * Nat.0.factorial
    factorial_zero
    Nat.0.factorial = Nat.1
    Nat.1.factorial = Nat.1 * Nat.1
    Nat.1 * Nat.1 = Nat.1
    Nat.2.factorial = Nat.1.suc * Nat.1
    Nat.1.suc * Nat.1 = Nat.2
    Nat.2.factorial = Nat.2
}

/// The factorial of three is six.
theorem factorial_three_nat {
    Nat.3.factorial = Nat.6
} by {
    factorial_step(Nat.2)
    Nat.2.suc.factorial = Nat.2.suc * Nat.2.factorial
    Nat.2.suc = Nat.3
    Nat.3.factorial = Nat.2.suc * Nat.2.factorial
    factorial_two_nat
    Nat.2.factorial = Nat.2
    Nat.3.factorial = Nat.2.suc * Nat.2
    Nat.2.suc * Nat.2 = Nat.6
    Nat.3.factorial = Nat.6
}

/// The image of the factorial of two in the reals is two.
theorem from_nat_factorial_two_real {
    from_nat[Real](Nat.2.factorial) = taylor_two
} by {
    factorial_two_nat
    Nat.2.factorial = Nat.2
    from_nat[Real](Nat.2.factorial) = from_nat[Real](Nat.2)
    from_nat_two_real
    from_nat[Real](Nat.2) = taylor_two
    from_nat[Real](Nat.2.factorial) = taylor_two
}

/// The image of the factorial of three in the reals is six.
theorem from_nat_factorial_three_real {
    from_nat[Real](Nat.3.factorial) = taylor_six
} by {
    factorial_three_nat
    Nat.3.factorial = Nat.6
    from_nat[Real](Nat.3.factorial) = from_nat[Real](Nat.6)
    from_nat_six_real
    from_nat[Real](Nat.6) = taylor_six
    from_nat[Real](Nat.3.factorial) = taylor_six
}

/// The image of the factorial of one in the reals is one.
theorem from_nat_factorial_one_real {
    from_nat[Real](Nat.1.factorial) = Real.1
} by {
    factorial_one
    Nat.1.factorial = Nat.1
    from_nat[Real](Nat.1.factorial) = from_nat[Real](Nat.1)
    from_nat_one[Real]
    from_nat[Real](Nat.1) = Real.1
    from_nat[Real](Nat.1.factorial) = Real.1
}

/// The image of the factorial of zero in the reals is one.
theorem from_nat_factorial_zero_real {
    from_nat[Real](Nat.0.factorial) = Real.1
} by {
    factorial_zero
    Nat.0.factorial = Nat.1
    from_nat[Real](Nat.0.factorial) = from_nat[Real](Nat.1)
    from_nat_one[Real]
    from_nat[Real](Nat.1) = Real.1
    from_nat[Real](Nat.0.factorial) = Real.1
}

/// The square of a real number is its product with itself.
theorem real_pow_two_eq_mul(x: Real) {
    x.pow(Nat.2) = x * x
} by {
    pow_suc(x, Nat.1)
    x.pow(Nat.1.suc) = x * x.pow(Nat.1)
    Nat.1.suc = Nat.2
    x.pow(Nat.2) = x * x.pow(Nat.1)
    pow_one[Real](x)
    x.pow(Nat.1) = x
    x.pow(Nat.2) = x * x
}

/// The cube of a real number is its triple product with itself.
theorem real_pow_three_eq_mul(x: Real) {
    x.pow(Nat.3) = x * x * x
} by {
    pow_suc(x, Nat.2)
    x.pow(Nat.2.suc) = x * x.pow(Nat.2)
    Nat.2.suc = Nat.3
    x.pow(Nat.3) = x * x.pow(Nat.2)
    real_pow_two_eq_mul(x)
    x.pow(Nat.2) = x * x
    x.pow(Nat.3) = x * (x * x)
    mul_assoc(x, x, x)
    x * (x * x) = x * x * x
    x.pow(Nat.3) = x * x * x
}

/// The first power of a real number is the number itself.
theorem real_pow_one_eq(x: Real) {
    x.pow(Nat.1) = x
} by {
    pow_one[Real](x)
    x.pow(Nat.1) = x
}

/// The zeroth power of a real number is one.
theorem real_pow_zero_eq(x: Real) {
    x.pow(Nat.0) = Real.1
} by {
    pow_zero[Real](x)
    x.pow(Nat.0) = Real.1
}

/// The fourth power of a real number is its square squared.
theorem real_pow_four_eq_mul(x: Real) {
    x.pow(Nat.4) = (x * x) * (x * x)
} by {
    pow_add(x, Nat.2, Nat.2)
    x.pow(Nat.2 + Nat.2) = x.pow(Nat.2) * x.pow(Nat.2)
    Nat.2 + Nat.2 = Nat.4
    x.pow(Nat.4) = x.pow(Nat.2) * x.pow(Nat.2)
    real_pow_two_eq_mul(x)
    x.pow(Nat.2) = x * x
    x.pow(Nat.4) = (x * x) * (x * x)
}

/// Dividing by one is the identity.
theorem real_div_one(a: Real) {
    a / Real.1 = a
} by {
    a / Real.1 = a * Real.1.inverse
    inverse_one[Real]
    Real.1.inverse = Real.1
    a * Real.1.inverse = a * Real.1
    mul_one_right(a)
    a * Real.1 = a
    a / Real.1 = a
}

/// A number divided by its square's reciprocal relations: a nonzero factor can
/// be cancelled from a product equality.
theorem real_mul_inverse_comm(a: Real, b: Real) {
    a != Real.0 implies (a * b) * a.inverse = b
} by {
    if a != Real.0 {
        mul_inverse(a)
        a * a.inverse = Real.1
        mul_assoc(a, b, a.inverse)
        (a * b) * a.inverse = a * (b * a.inverse)
        real_mul_comm(b, a.inverse)
        b * a.inverse = a.inverse * b
        a * (b * a.inverse) = a * (a.inverse * b)
        mul_assoc(a, a.inverse, b)
        a * (a.inverse * b) = (a * a.inverse) * b
        (a * a.inverse) * b = Real.1 * b
        mul_one_left(b)
        Real.1 * b = b
        a * (a.inverse * b) = b
        a * (b * a.inverse) = b
        (a * b) * a.inverse = b
    }
}

/// The partial sum of the first two terms.
theorem partial_two_sum(f: Nat -> Real) {
    partial(f, Nat.2) = f(Nat.0) + f(Nat.1)
} by {
    partial_split_last[Real](f, Nat.1)
    partial(f, Nat.1.suc) = partial(f, Nat.1) + f(Nat.1)
    Nat.1.suc = Nat.2
    partial(f, Nat.2) = partial(f, Nat.1) + f(Nat.1)
    partial_one[Real](f)
    partial(f, Nat.1) = f(Nat.0)
    partial(f, Nat.2) = f(Nat.0) + f(Nat.1)
}

/// The partial sum of the first three terms.
theorem partial_three_sum(f: Nat -> Real) {
    partial(f, Nat.3) = f(Nat.0) + f(Nat.1) + f(Nat.2)
} by {
    partial_split_last[Real](f, Nat.2)
    partial(f, Nat.2.suc) = partial(f, Nat.2) + f(Nat.2)
    Nat.2.suc = Nat.3
    partial(f, Nat.3) = partial(f, Nat.2) + f(Nat.2)
    partial_two_sum(f)
    partial(f, Nat.2) = f(Nat.0) + f(Nat.1)
    partial(f, Nat.3) = (f(Nat.0) + f(Nat.1)) + f(Nat.2)
    f(Nat.0) + f(Nat.1) + f(Nat.2) = (f(Nat.0) + f(Nat.1)) + f(Nat.2)
    partial(f, Nat.3) = f(Nat.0) + f(Nat.1) + f(Nat.2)
}

/// The partial sum of the first four terms.
theorem partial_four_sum(f: Nat -> Real) {
    partial(f, Nat.4) = f(Nat.0) + f(Nat.1) + f(Nat.2) + f(Nat.3)
} by {
    partial_split_last[Real](f, Nat.3)
    partial(f, Nat.3.suc) = partial(f, Nat.3) + f(Nat.3)
    Nat.3.suc = Nat.4
    partial(f, Nat.4) = partial(f, Nat.3) + f(Nat.3)
    partial_three_sum(f)
    partial(f, Nat.3) = f(Nat.0) + f(Nat.1) + f(Nat.2)
    partial(f, Nat.4) = (f(Nat.0) + f(Nat.1) + f(Nat.2)) + f(Nat.3)
    f(Nat.0) + f(Nat.1) + f(Nat.2) + f(Nat.3) =
        (f(Nat.0) + f(Nat.1) + f(Nat.2)) + f(Nat.3)
    partial(f, Nat.4) = f(Nat.0) + f(Nat.1) + f(Nat.2) + f(Nat.3)
}

/// Taylor's theorem of order zero with Lagrange remainder: the mean value
/// theorem, restated in the iterated-derivative notation.
theorem taylor_general_zero(f: Real -> Real, df: Real -> Real, a: Real, b: Real) {
    a < b and is_derivative_fn(f, df)
    implies exists(c: Real) {
        a < c and c < b and
        f(b) = taylor_poly(f, taylor_chain1(f, df), a, b, Nat.0) +
            taylor_remainder(f, taylor_chain1(f, df), a, b, c, Nat.0)
    }
} by {
    if a < b and is_derivative_fn(f, df) {
        forall(x: Real) {
            is_derivative_fn_iff(f, df)
            is_derivative_fn(f, df) = forall(y: Real) {
                has_derivative_at(f, y, df(y))
            }
            forall(y: Real) {
                has_derivative_at(f, y, df(y))
            }
            has_derivative_at(f, x, df(x))
            is_derivative_fn_imp_continuous_at(f, df, x)
            continuous_at(f, x)
        }
        continuous(f) = forall(x: Real) {
            continuous_at(f, x)
        }
        continuous(f)
        taylor_order_one(f, df, a, b)
        let c: Real satisfy {
            a < c and c < b and f(b) = f(a) + df(c) * (b - a)
        }
        a < c and c < b
        f(b) = f(a) + df(c) * (b - a)
        taylor_poly(f, taylor_chain1(f, df), a, b, Nat.0) =
            partial(taylor_term(f, taylor_chain1(f, df), a, b), Nat.1)
        partial_one[Real](taylor_term(f, taylor_chain1(f, df), a, b))
        partial(taylor_term(f, taylor_chain1(f, df), a, b), Nat.1) =
            taylor_term(f, taylor_chain1(f, df), a, b, Nat.0)
        taylor_poly(f, taylor_chain1(f, df), a, b, Nat.0) =
            taylor_term(f, taylor_chain1(f, df), a, b, Nat.0)
        taylor_term(f, taylor_chain1(f, df), a, b, Nat.0) =
            (iterated_derivative(f, taylor_chain1(f, df), Nat.0, a) /
                from_nat[Real](Nat.0.factorial)) * (b - a).pow(Nat.0)
        iterated_derivative(f, taylor_chain1(f, df), Nat.0, a) =
            taylor_chain1(f, df, Nat.0, a)
        taylor_chain1_zero(f, df)
        taylor_chain1(f, df, Nat.0) = f
        taylor_chain1(f, df, Nat.0, a) = f(a)
        iterated_derivative(f, taylor_chain1(f, df), Nat.0, a) = f(a)
        from_nat_factorial_zero_real
        from_nat[Real](Nat.0.factorial) = Real.1
        real_pow_zero_eq(b - a)
        (b - a).pow(Nat.0) = Real.1
        taylor_term(f, taylor_chain1(f, df), a, b, Nat.0) = (f(a) / Real.1) * Real.1
        real_div_one(f(a))
        f(a) / Real.1 = f(a)
        (f(a) / Real.1) * Real.1 = f(a) * Real.1
        mul_one_right(f(a))
        f(a) * Real.1 = f(a)
        (f(a) / Real.1) * Real.1 = f(a)
        taylor_term(f, taylor_chain1(f, df), a, b, Nat.0) = f(a)
        taylor_poly(f, taylor_chain1(f, df), a, b, Nat.0) = f(a)
        taylor_remainder(f, taylor_chain1(f, df), a, b, c, Nat.0) =
            (iterated_derivative(f, taylor_chain1(f, df), Nat.0.suc, c) /
                from_nat[Real](Nat.0.suc.factorial)) * (b - a).pow(Nat.0.suc)
        Nat.0.suc = Nat.1
        taylor_remainder(f, taylor_chain1(f, df), a, b, c, Nat.0) =
            (iterated_derivative(f, taylor_chain1(f, df), Nat.1, c) /
                from_nat[Real](Nat.1.factorial)) * (b - a).pow(Nat.1)
        iterated_derivative(f, taylor_chain1(f, df), Nat.1, c) =
            taylor_chain1(f, df, Nat.1, c)
        taylor_chain1_suc(f, df, Nat.0)
        taylor_chain1(f, df, Nat.0.suc) = df
        Nat.0.suc = Nat.1
        taylor_chain1(f, df, Nat.1) = df
        taylor_chain1(f, df, Nat.1, c) = df(c)
        iterated_derivative(f, taylor_chain1(f, df), Nat.1, c) = df(c)
        from_nat_factorial_one_real
        from_nat[Real](Nat.1.factorial) = Real.1
        real_pow_one_eq(b - a)
        (b - a).pow(Nat.1) = b - a
        taylor_remainder(f, taylor_chain1(f, df), a, b, c, Nat.0) =
            (df(c) / Real.1) * (b - a)
        real_div_one(df(c))
        df(c) / Real.1 = df(c)
        (df(c) / Real.1) * (b - a) = df(c) * (b - a)
        taylor_remainder(f, taylor_chain1(f, df), a, b, c, Nat.0) = df(c) * (b - a)
        taylor_poly(f, taylor_chain1(f, df), a, b, Nat.0) +
            taylor_remainder(f, taylor_chain1(f, df), a, b, c, Nat.0) =
            f(a) + df(c) * (b - a)
        f(b) = taylor_poly(f, taylor_chain1(f, df), a, b, Nat.0) +
            taylor_remainder(f, taylor_chain1(f, df), a, b, c, Nat.0)
        exists(c2: Real) {
            a < c2 and c2 < b and
            f(b) = taylor_poly(f, taylor_chain1(f, df), a, b, Nat.0) +
                taylor_remainder(f, taylor_chain1(f, df), a, b, c2, Nat.0)
        }
    }
}

/// Taylor's theorem of order one with Lagrange remainder, restated in the
/// iterated-derivative notation: this is the second-order theorem of taylor.ac
/// with the second-order polynomial written as a sum of Taylor terms.
theorem taylor_general_one(f: Real -> Real, df: Real -> Real, ddf: Real -> Real, a: Real, b: Real) {
    a < b and is_derivative_fn(f, df) and is_derivative_fn(df, ddf)
    implies exists(c: Real) {
        a < c and c < b and
        f(b) = taylor_poly(f, taylor_chain2(f, df, ddf), a, b, Nat.1) +
            taylor_remainder(f, taylor_chain2(f, df, ddf), a, b, c, Nat.1)
    }
} by {
    if a < b and is_derivative_fn(f, df) and is_derivative_fn(df, ddf) {
        taylor_order_two(f, df, ddf, a, b)
        let c: Real satisfy {
            a < c and c < b and
            f(b) = f(a) + df(a) * (b - a) + (ddf(c) / taylor_two) * (b - a) * (b - a)
        }
        a < c and c < b
        f(b) = f(a) + df(a) * (b - a) + (ddf(c) / taylor_two) * (b - a) * (b - a)
        taylor_poly(f, taylor_chain2(f, df, ddf), a, b, Nat.1) =
            partial(taylor_term(f, taylor_chain2(f, df, ddf), a, b), Nat.2)
        partial_two_sum(taylor_term(f, taylor_chain2(f, df, ddf), a, b))
        partial(taylor_term(f, taylor_chain2(f, df, ddf), a, b), Nat.2) =
            taylor_term(f, taylor_chain2(f, df, ddf), a, b, Nat.0) +
            taylor_term(f, taylor_chain2(f, df, ddf), a, b, Nat.1)
        taylor_poly(f, taylor_chain2(f, df, ddf), a, b, Nat.1) =
            taylor_term(f, taylor_chain2(f, df, ddf), a, b, Nat.0) +
            taylor_term(f, taylor_chain2(f, df, ddf), a, b, Nat.1)
        taylor_term(f, taylor_chain2(f, df, ddf), a, b, Nat.0) =
            (iterated_derivative(f, taylor_chain2(f, df, ddf), Nat.0, a) /
                from_nat[Real](Nat.0.factorial)) * (b - a).pow(Nat.0)
        iterated_derivative(f, taylor_chain2(f, df, ddf), Nat.0, a) =
            taylor_chain2(f, df, ddf, Nat.0, a)
        taylor_chain2_zero(f, df, ddf)
        taylor_chain2(f, df, ddf, Nat.0) = f
        taylor_chain2(f, df, ddf, Nat.0, a) = f(a)
        iterated_derivative(f, taylor_chain2(f, df, ddf), Nat.0, a) = f(a)
        from_nat_factorial_zero_real
        from_nat[Real](Nat.0.factorial) = Real.1
        real_pow_zero_eq(b - a)
        (b - a).pow(Nat.0) = Real.1
        taylor_term(f, taylor_chain2(f, df, ddf), a, b, Nat.0) = (f(a) / Real.1) * Real.1
        real_div_one(f(a))
        f(a) / Real.1 = f(a)
        (f(a) / Real.1) * Real.1 = f(a) * Real.1
        mul_one_right(f(a))
        f(a) * Real.1 = f(a)
        (f(a) / Real.1) * Real.1 = f(a)
        taylor_term(f, taylor_chain2(f, df, ddf), a, b, Nat.0) = f(a)
        taylor_term(f, taylor_chain2(f, df, ddf), a, b, Nat.1) =
            (iterated_derivative(f, taylor_chain2(f, df, ddf), Nat.1, a) /
                from_nat[Real](Nat.1.factorial)) * (b - a).pow(Nat.1)
        iterated_derivative(f, taylor_chain2(f, df, ddf), Nat.1, a) =
            taylor_chain2(f, df, ddf, Nat.1, a)
        taylor_chain2_one(f, df, ddf)
        taylor_chain2(f, df, ddf, Nat.1) = df
        taylor_chain2(f, df, ddf, Nat.1, a) = df(a)
        iterated_derivative(f, taylor_chain2(f, df, ddf), Nat.1, a) = df(a)
        from_nat_factorial_one_real
        from_nat[Real](Nat.1.factorial) = Real.1
        real_pow_one_eq(b - a)
        (b - a).pow(Nat.1) = b - a
        taylor_term(f, taylor_chain2(f, df, ddf), a, b, Nat.1) =
            (df(a) / Real.1) * (b - a)
        real_div_one(df(a))
        df(a) / Real.1 = df(a)
        (df(a) / Real.1) * (b - a) = df(a) * (b - a)
        taylor_term(f, taylor_chain2(f, df, ddf), a, b, Nat.1) = df(a) * (b - a)
        taylor_poly(f, taylor_chain2(f, df, ddf), a, b, Nat.1) =
            f(a) + df(a) * (b - a)
        taylor_remainder(f, taylor_chain2(f, df, ddf), a, b, c, Nat.1) =
            (iterated_derivative(f, taylor_chain2(f, df, ddf), Nat.1.suc, c) /
                from_nat[Real](Nat.1.suc.factorial)) * (b - a).pow(Nat.1.suc)
        Nat.1.suc = Nat.2
        taylor_remainder(f, taylor_chain2(f, df, ddf), a, b, c, Nat.1) =
            (iterated_derivative(f, taylor_chain2(f, df, ddf), Nat.2, c) /
                from_nat[Real](Nat.2.factorial)) * (b - a).pow(Nat.2)
        iterated_derivative(f, taylor_chain2(f, df, ddf), Nat.2, c) =
            taylor_chain2(f, df, ddf, Nat.2, c)
        taylor_chain2_suc_suc(f, df, ddf, Nat.0)
        taylor_chain2(f, df, ddf, Nat.0.suc.suc) = ddf
        Nat.0.suc.suc = Nat.2
        taylor_chain2(f, df, ddf, Nat.2) = ddf
        taylor_chain2(f, df, ddf, Nat.2, c) = ddf(c)
        iterated_derivative(f, taylor_chain2(f, df, ddf), Nat.2, c) = ddf(c)
        from_nat_factorial_two_real
        from_nat[Real](Nat.2.factorial) = taylor_two
        real_pow_two_eq_mul(b - a)
        (b - a).pow(Nat.2) = (b - a) * (b - a)
        taylor_remainder(f, taylor_chain2(f, df, ddf), a, b, c, Nat.1) =
            (ddf(c) / taylor_two) * ((b - a) * (b - a))
        (ddf(c) / taylor_two) * ((b - a) * (b - a)) =
            (ddf(c) / taylor_two) * (b - a) * (b - a)
        taylor_remainder(f, taylor_chain2(f, df, ddf), a, b, c, Nat.1) =
            (ddf(c) / taylor_two) * (b - a) * (b - a)
        taylor_poly(f, taylor_chain2(f, df, ddf), a, b, Nat.1) +
            taylor_remainder(f, taylor_chain2(f, df, ddf), a, b, c, Nat.1) =
            f(a) + df(a) * (b - a) + (ddf(c) / taylor_two) * (b - a) * (b - a)
        f(b) = taylor_poly(f, taylor_chain2(f, df, ddf), a, b, Nat.1) +
            taylor_remainder(f, taylor_chain2(f, df, ddf), a, b, c, Nat.1)
        exists(c2: Real) {
            a < c2 and c2 < b and
            f(b) = taylor_poly(f, taylor_chain2(f, df, ddf), a, b, Nat.1) +
                taylor_remainder(f, taylor_chain2(f, df, ddf), a, b, c2, Nat.1)
        }
    }
}

/// The doubled second-order increment of f over [a, b]: twice the value at b
/// minus the second-order Taylor polynomial of f at a.
define taylor3_doubled_delta(f: Real -> Real, df: Real -> Real, ddf: Real -> Real, a: Real, b: Real) -> Real {
    taylor_two * f(b) - taylor_two * f(a) - taylor_two * df(a) * (b - a) -
        ddf(a) * (b - a).pow(Nat.2)
}

/// The coefficient of the cubic remainder term: the doubled second-order
/// increment divided by (b - a)^3.
define taylor3_remainder_coeff(f: Real -> Real, df: Real -> Real, ddf: Real -> Real, a: Real, b: Real) -> Real {
    taylor3_doubled_delta(f, df, ddf, a, b) / (b - a).pow(Nat.3)
}

/// The auxiliary function for the third-order Taylor theorem, scaled by two:
/// the value of the truncated expansion at x plus the cubic remainder term,
/// subtracted from twice f(b).  It vanishes at a and at b, so Rolle's theorem
/// applies to it.
define taylor3_aux(f: Real -> Real, df: Real -> Real, ddf: Real -> Real, a: Real, b: Real, x: Real) -> Real {
    taylor_two * f(b) - taylor_two * f(x) - taylor_two * df(x) * (b - x) -
        ddf(x) * (b - x).pow(Nat.2) -
        taylor3_remainder_coeff(f, df, ddf, a, b) * (b - x).pow(Nat.3)
}

/// The derivative of the auxiliary function of the third-order Taylor theorem.
define taylor3_aux_derivative(f: Real -> Real, df: Real -> Real, ddf: Real -> Real, dddf: Real -> Real, a: Real, b: Real, x: Real) -> Real {
    (b - x).pow(Nat.2) * (taylor_three * taylor3_remainder_coeff(f, df, ddf, a, b) - dddf(x))
}

/// The auxiliary function of the third-order Taylor theorem as a pointwise
/// combination of f, df, ddf, constants, an affine function, the square and the
/// cube.
define taylor3_aux_pointwise(f: Real -> Real, df: Real -> Real, ddf: Real -> Real, a: Real, b: Real) -> Real -> Real {
    pointwise_add(
        pointwise_add(
            pointwise_add(
                pointwise_add(constant[Real, Real](taylor_two * f(b)),
                    pointwise_neg(pointwise_mul(constant[Real, Real](taylor_two), f))),
                pointwise_neg(pointwise_mul(
                    pointwise_mul(constant[Real, Real](taylor_two), df),
                    affine_real(-Real.1, b)))),
            pointwise_neg(pointwise_mul(ddf,
                compose(square_real, affine_real(-Real.1, b))))),
        pointwise_neg(pointwise_mul(
            constant[Real, Real](taylor3_remainder_coeff(f, df, ddf, a, b)),
            compose(cube_real, affine_real(-Real.1, b)))))
}
/// The auxiliary function of the third-order Taylor theorem agrees with its
/// pointwise combination.
theorem taylor3_aux_eq_pointwise(f: Real -> Real, df: Real -> Real, ddf: Real -> Real, a: Real, b: Real) {
    taylor3_aux(f, df, ddf, a, b) = taylor3_aux_pointwise(f, df, ddf, a, b)
} by {
    forall(x: Real) {
        taylor3_aux(f, df, ddf, a, b, x) = taylor_two * f(b) - taylor_two * f(x) -
            taylor_two * df(x) * (b - x) - ddf(x) * (b - x).pow(Nat.2) -
            taylor3_remainder_coeff(f, df, ddf, a, b) * (b - x).pow(Nat.3)
        constant[Real, Real](taylor_two * f(b), x) = taylor_two * f(b)
        pointwise_mul(constant[Real, Real](taylor_two), f, x) =
            constant[Real, Real](taylor_two, x) * f(x)
        constant[Real, Real](taylor_two, x) = taylor_two
        pointwise_mul(constant[Real, Real](taylor_two), f, x) = taylor_two * f(x)
        pointwise_neg(pointwise_mul(constant[Real, Real](taylor_two), f), x) =
            -(pointwise_mul(constant[Real, Real](taylor_two), f, x))
        pointwise_neg(pointwise_mul(constant[Real, Real](taylor_two), f), x) =
            -(taylor_two * f(x))
        pointwise_mul(constant[Real, Real](taylor_two), df, x) =
            constant[Real, Real](taylor_two, x) * df(x)
        pointwise_mul(constant[Real, Real](taylor_two), df, x) = taylor_two * df(x)
        affine_real(-Real.1, b, x) = -Real.1 * x + b
        mul_neg_left(Real.1, x)
        -Real.1 * x = -(Real.1 * x)
        mul_one_left(x)
        Real.1 * x = x
        -(Real.1 * x) = -x
        -Real.1 * x = -x
        -x + b = b - x
        affine_real(-Real.1, b, x) = b - x
        pointwise_mul(pointwise_mul(constant[Real, Real](taylor_two), df), affine_real(-Real.1, b), x) =
            pointwise_mul(constant[Real, Real](taylor_two), df, x) *
                affine_real(-Real.1, b, x)
        pointwise_mul(pointwise_mul(constant[Real, Real](taylor_two), df), affine_real(-Real.1, b), x) = taylor_two * df(x) * (b - x)
        pointwise_neg(pointwise_mul(pointwise_mul(constant[Real, Real](taylor_two), df), affine_real(-Real.1, b)), x) =
            -(pointwise_mul(pointwise_mul(constant[Real, Real](taylor_two), df), affine_real(-Real.1, b), x))
        pointwise_neg(pointwise_mul(pointwise_mul(constant[Real, Real](taylor_two), df), affine_real(-Real.1, b)), x) = -(taylor_two * df(x) * (b - x))
        compose(square_real, affine_real(-Real.1, b), x) =
            square_real(affine_real(-Real.1, b, x))
        square_real(affine_real(-Real.1, b, x)) = square_real(b - x)
        real_pow_two_eq_mul(b - x)
        (b - x).pow(Nat.2) = (b - x) * (b - x)
        square_real(b - x) = (b - x).pow(Nat.2)
        compose(square_real, affine_real(-Real.1, b), x) = (b - x).pow(Nat.2)
        pointwise_mul(ddf, compose(square_real, affine_real(-Real.1, b)), x) =
            ddf(x) * compose(square_real, affine_real(-Real.1, b), x)
        pointwise_mul(ddf, compose(square_real, affine_real(-Real.1, b)), x) =
            ddf(x) * (b - x).pow(Nat.2)
        pointwise_neg(pointwise_mul(ddf, compose(square_real, affine_real(-Real.1, b))), x) =
            -(pointwise_mul(ddf, compose(square_real, affine_real(-Real.1, b)), x))
        pointwise_neg(pointwise_mul(ddf, compose(square_real, affine_real(-Real.1, b))), x) =
            -(ddf(x) * (b - x).pow(Nat.2))
        compose(cube_real, affine_real(-Real.1, b), x) =
            cube_real(affine_real(-Real.1, b, x))
        cube_real(affine_real(-Real.1, b, x)) = cube_real(b - x)
        real_pow_three_eq_mul(b - x)
        (b - x).pow(Nat.3) = (b - x) * (b - x) * (b - x)
        cube_real(b - x) = (b - x).pow(Nat.3)
        compose(cube_real, affine_real(-Real.1, b), x) = (b - x).pow(Nat.3)
        pointwise_mul(constant[Real, Real](taylor3_remainder_coeff(f, df, ddf, a, b)),
            compose(cube_real, affine_real(-Real.1, b)), x) =
            constant[Real, Real](taylor3_remainder_coeff(f, df, ddf, a, b), x) *
                compose(cube_real, affine_real(-Real.1, b), x)
        constant[Real, Real](taylor3_remainder_coeff(f, df, ddf, a, b), x) =
            taylor3_remainder_coeff(f, df, ddf, a, b)
        pointwise_mul(constant[Real, Real](taylor3_remainder_coeff(f, df, ddf, a, b)),
            compose(cube_real, affine_real(-Real.1, b)), x) =
            taylor3_remainder_coeff(f, df, ddf, a, b) * (b - x).pow(Nat.3)
        pointwise_neg(pointwise_mul(constant[Real, Real](taylor3_remainder_coeff(f, df, ddf, a, b)),
            compose(cube_real, affine_real(-Real.1, b))), x) =
            -(pointwise_mul(constant[Real, Real](taylor3_remainder_coeff(f, df, ddf, a, b)),
                compose(cube_real, affine_real(-Real.1, b)), x))
        pointwise_neg(pointwise_mul(constant[Real, Real](taylor3_remainder_coeff(f, df, ddf, a, b)),
            compose(cube_real, affine_real(-Real.1, b))), x) =
            -(taylor3_remainder_coeff(f, df, ddf, a, b) * (b - x).pow(Nat.3))
        pointwise_add(constant[Real, Real](taylor_two * f(b)), pointwise_neg(pointwise_mul(constant[Real, Real](taylor_two), f)), x) =
            constant[Real, Real](taylor_two * f(b), x) + pointwise_neg(pointwise_mul(constant[Real, Real](taylor_two), f), x)
        pointwise_add(constant[Real, Real](taylor_two * f(b)), pointwise_neg(pointwise_mul(constant[Real, Real](taylor_two), f)), x) = taylor_two * f(b) + -(taylor_two * f(x))
        taylor_two * f(b) + -(taylor_two * f(x)) = taylor_two * f(b) - taylor_two * f(x)
        pointwise_add(pointwise_add(constant[Real, Real](taylor_two * f(b)), pointwise_neg(pointwise_mul(constant[Real, Real](taylor_two), f))), pointwise_neg(pointwise_mul(pointwise_mul(constant[Real, Real](taylor_two), df), affine_real(-Real.1, b))), x) =
            pointwise_add(constant[Real, Real](taylor_two * f(b)), pointwise_neg(pointwise_mul(constant[Real, Real](taylor_two), f)), x) + pointwise_neg(pointwise_mul(pointwise_mul(constant[Real, Real](taylor_two), df), affine_real(-Real.1, b)), x)
        pointwise_add(pointwise_add(constant[Real, Real](taylor_two * f(b)), pointwise_neg(pointwise_mul(constant[Real, Real](taylor_two), f))), pointwise_neg(pointwise_mul(pointwise_mul(constant[Real, Real](taylor_two), df), affine_real(-Real.1, b))), x) = (taylor_two * f(b) - taylor_two * f(x)) + -(taylor_two * df(x) * (b - x))
        (taylor_two * f(b) - taylor_two * f(x)) + -(taylor_two * df(x) * (b - x)) = taylor_two * f(b) - taylor_two * f(x) - taylor_two * df(x) * (b - x)
        pointwise_add(pointwise_add(pointwise_add(constant[Real, Real](taylor_two * f(b)), pointwise_neg(pointwise_mul(constant[Real, Real](taylor_two), f))), pointwise_neg(pointwise_mul(pointwise_mul(constant[Real, Real](taylor_two), df), affine_real(-Real.1, b)))), pointwise_neg(pointwise_mul(ddf, compose(square_real, affine_real(-Real.1, b)))), x) =
            pointwise_add(pointwise_add(constant[Real, Real](taylor_two * f(b)), pointwise_neg(pointwise_mul(constant[Real, Real](taylor_two), f))), pointwise_neg(pointwise_mul(pointwise_mul(constant[Real, Real](taylor_two), df), affine_real(-Real.1, b))), x) + pointwise_neg(pointwise_mul(ddf, compose(square_real, affine_real(-Real.1, b))), x)
        pointwise_add(pointwise_add(pointwise_add(constant[Real, Real](taylor_two * f(b)), pointwise_neg(pointwise_mul(constant[Real, Real](taylor_two), f))), pointwise_neg(pointwise_mul(pointwise_mul(constant[Real, Real](taylor_two), df), affine_real(-Real.1, b)))), pointwise_neg(pointwise_mul(ddf, compose(square_real, affine_real(-Real.1, b)))), x) = (taylor_two * f(b) - taylor_two * f(x) - taylor_two * df(x) * (b - x)) + -(ddf(x) * (b - x).pow(Nat.2))
        (taylor_two * f(b) - taylor_two * f(x) - taylor_two * df(x) * (b - x)) + -(ddf(x) * (b - x).pow(Nat.2)) = taylor_two * f(b) - taylor_two * f(x) - taylor_two * df(x) * (b - x) - ddf(x) * (b - x).pow(Nat.2)
        pointwise_add(pointwise_add(pointwise_add(pointwise_add(constant[Real, Real](taylor_two * f(b)), pointwise_neg(pointwise_mul(constant[Real, Real](taylor_two), f))), pointwise_neg(pointwise_mul(pointwise_mul(constant[Real, Real](taylor_two), df), affine_real(-Real.1, b)))), pointwise_neg(pointwise_mul(ddf, compose(square_real, affine_real(-Real.1, b))))), pointwise_neg(pointwise_mul(constant[Real, Real](taylor3_remainder_coeff(f, df, ddf, a, b)), compose(cube_real, affine_real(-Real.1, b)))), x) =
            pointwise_add(pointwise_add(pointwise_add(constant[Real, Real](taylor_two * f(b)), pointwise_neg(pointwise_mul(constant[Real, Real](taylor_two), f))), pointwise_neg(pointwise_mul(pointwise_mul(constant[Real, Real](taylor_two), df), affine_real(-Real.1, b)))), pointwise_neg(pointwise_mul(ddf, compose(square_real, affine_real(-Real.1, b)))), x) + pointwise_neg(pointwise_mul(constant[Real, Real](taylor3_remainder_coeff(f, df, ddf, a, b)), compose(cube_real, affine_real(-Real.1, b))), x)
        pointwise_add(pointwise_add(pointwise_add(pointwise_add(constant[Real, Real](taylor_two * f(b)), pointwise_neg(pointwise_mul(constant[Real, Real](taylor_two), f))), pointwise_neg(pointwise_mul(pointwise_mul(constant[Real, Real](taylor_two), df), affine_real(-Real.1, b)))), pointwise_neg(pointwise_mul(ddf, compose(square_real, affine_real(-Real.1, b))))), pointwise_neg(pointwise_mul(constant[Real, Real](taylor3_remainder_coeff(f, df, ddf, a, b)), compose(cube_real, affine_real(-Real.1, b)))), x) = (taylor_two * f(b) - taylor_two * f(x) - taylor_two * df(x) * (b - x) - ddf(x) * (b - x).pow(Nat.2)) + -(taylor3_remainder_coeff(f, df, ddf, a, b) * (b - x).pow(Nat.3))
        (taylor_two * f(b) - taylor_two * f(x) - taylor_two * df(x) * (b - x) - ddf(x) * (b - x).pow(Nat.2)) + -(taylor3_remainder_coeff(f, df, ddf, a, b) * (b - x).pow(Nat.3)) = taylor_two * f(b) - taylor_two * f(x) - taylor_two * df(x) * (b - x) - ddf(x) * (b - x).pow(Nat.2) - taylor3_remainder_coeff(f, df, ddf, a, b) * (b - x).pow(Nat.3)
        pointwise_add(pointwise_add(pointwise_add(pointwise_add(constant[Real, Real](taylor_two * f(b)), pointwise_neg(pointwise_mul(constant[Real, Real](taylor_two), f))), pointwise_neg(pointwise_mul(pointwise_mul(constant[Real, Real](taylor_two), df), affine_real(-Real.1, b)))), pointwise_neg(pointwise_mul(ddf, compose(square_real, affine_real(-Real.1, b))))), pointwise_neg(pointwise_mul(constant[Real, Real](taylor3_remainder_coeff(f, df, ddf, a, b)), compose(cube_real, affine_real(-Real.1, b)))), x) = taylor_two * f(b) - taylor_two * f(x) - taylor_two * df(x) * (b - x) - ddf(x) * (b - x).pow(Nat.2) - taylor3_remainder_coeff(f, df, ddf, a, b) * (b - x).pow(Nat.3)
        taylor3_aux(f, df, ddf, a, b, x) = pointwise_add(pointwise_add(pointwise_add(pointwise_add(constant[Real, Real](taylor_two * f(b)), pointwise_neg(pointwise_mul(constant[Real, Real](taylor_two), f))), pointwise_neg(pointwise_mul(pointwise_mul(constant[Real, Real](taylor_two), df), affine_real(-Real.1, b)))), pointwise_neg(pointwise_mul(ddf, compose(square_real, affine_real(-Real.1, b))))), pointwise_neg(pointwise_mul(constant[Real, Real](taylor3_remainder_coeff(f, df, ddf, a, b)), compose(cube_real, affine_real(-Real.1, b)))), x)
    }
    function_extensionality(taylor3_aux(f, df, ddf, a, b), taylor3_aux_pointwise(f, df, ddf, a, b))
    taylor3_aux(f, df, ddf, a, b) = taylor3_aux_pointwise(f, df, ddf, a, b)
}
/// The auxiliary function of the third-order Taylor theorem vanishes at a.
theorem taylor3_aux_at_a_zero(f: Real -> Real, df: Real -> Real, ddf: Real -> Real, a: Real, b: Real) {
    a < b implies taylor3_aux(f, df, ddf, a, b, a) = Real.0
} by {
    if a < b {
        taylor3_aux(f, df, ddf, a, b, a) = taylor_two * f(b) - taylor_two * f(a) -
            taylor_two * df(a) * (b - a) - ddf(a) * (b - a).pow(Nat.2) -
            taylor3_remainder_coeff(f, df, ddf, a, b) * (b - a).pow(Nat.3)
        lt_imp_ne_symm(a, b)
        b != a
        sub_ne_zero_of_ne(b, a)
        b - a != Real.0
        field_square_nonzero[Real](b - a)
        (b - a) * (b - a) != Real.0
        mul_not_zero[Real](b - a, (b - a) * (b - a))
        (b - a) * ((b - a) * (b - a)) != Real.0
        real_pow_three_eq_mul(b - a)
        (b - a).pow(Nat.3) = (b - a) * (b - a) * (b - a)
        (b - a).pow(Nat.3) != Real.0
        taylor3_remainder_coeff(f, df, ddf, a, b) =
            taylor3_doubled_delta(f, df, ddf, a, b) / (b - a).pow(Nat.3)
        div_mul_cancel_denominator(taylor3_doubled_delta(f, df, ddf, a, b), (b - a).pow(Nat.3))
        (taylor3_doubled_delta(f, df, ddf, a, b) / (b - a).pow(Nat.3)) * (b - a).pow(Nat.3) =
            taylor3_doubled_delta(f, df, ddf, a, b)
        taylor3_remainder_coeff(f, df, ddf, a, b) * (b - a).pow(Nat.3) =
            taylor3_doubled_delta(f, df, ddf, a, b)
        taylor3_doubled_delta(f, df, ddf, a, b) =
            taylor_two * f(b) - taylor_two * f(a) - taylor_two * df(a) * (b - a) -
                ddf(a) * (b - a).pow(Nat.2)
        taylor3_aux(f, df, ddf, a, b, a) = taylor_two * f(b) - taylor_two * f(a) -
            taylor_two * df(a) * (b - a) - ddf(a) * (b - a).pow(Nat.2) -
            taylor3_doubled_delta(f, df, ddf, a, b)
        taylor_two * f(b) - taylor_two * f(a) - taylor_two * df(a) * (b - a) -
            ddf(a) * (b - a).pow(Nat.2) -
            (taylor_two * f(b) - taylor_two * f(a) - taylor_two * df(a) * (b - a) -
                ddf(a) * (b - a).pow(Nat.2)) = Real.0
        taylor3_aux(f, df, ddf, a, b, a) = Real.0
    }
}

/// The auxiliary function of the third-order Taylor theorem vanishes at b.
theorem taylor3_aux_at_b_zero(f: Real -> Real, df: Real -> Real, ddf: Real -> Real, a: Real, b: Real) {
    taylor3_aux(f, df, ddf, a, b, b) = Real.0
} by {
    taylor3_aux(f, df, ddf, a, b, b) = taylor_two * f(b) - taylor_two * f(b) -
        taylor_two * df(b) * (b - b) - ddf(b) * (b - b).pow(Nat.2) -
        taylor3_remainder_coeff(f, df, ddf, a, b) * (b - b).pow(Nat.3)
    b - b = Real.0
    real_pow_two_eq_mul(b - b)
    (b - b).pow(Nat.2) = (b - b) * (b - b)
    (b - b) * (b - b) = Real.0 * Real.0
    mul_zero_left(Real.0)
    Real.0 * Real.0 = Real.0
    (b - b).pow(Nat.2) = Real.0
    real_pow_three_eq_mul(b - b)
    (b - b).pow(Nat.3) = (b - b) * (b - b) * (b - b)
    (b - b) * (b - b) * (b - b) = Real.0
    (b - b).pow(Nat.3) = Real.0
    taylor_two * df(b) * (b - b) = taylor_two * df(b) * Real.0
    mul_zero_right(taylor_two * df(b))
    (taylor_two * df(b)) * Real.0 = Real.0
    taylor_two * df(b) * (b - b) = Real.0
    ddf(b) * (b - b).pow(Nat.2) = ddf(b) * Real.0
    mul_zero_right(ddf(b))
    ddf(b) * Real.0 = Real.0
    ddf(b) * (b - b).pow(Nat.2) = Real.0
    taylor3_remainder_coeff(f, df, ddf, a, b) * (b - b).pow(Nat.3) =
        (taylor3_remainder_coeff(f, df, ddf, a, b)) * Real.0
    mul_zero_right(taylor3_remainder_coeff(f, df, ddf, a, b))
    (taylor3_remainder_coeff(f, df, ddf, a, b)) * Real.0 = Real.0
    taylor3_remainder_coeff(f, df, ddf, a, b) * (b - b).pow(Nat.3) = Real.0
    taylor_two * f(b) - taylor_two * f(b) = Real.0
    taylor_two * f(b) - taylor_two * f(b) - Real.0 - Real.0 - Real.0 = Real.0
    taylor3_aux(f, df, ddf, a, b, b) = Real.0
}

/// The auxiliary function of the third-order Taylor theorem takes equal values
/// at the endpoints of the interval.
theorem taylor3_aux_endpoints_equal(f: Real -> Real, df: Real -> Real, ddf: Real -> Real, a: Real, b: Real) {
    a < b implies taylor3_aux(f, df, ddf, a, b, a) = taylor3_aux(f, df, ddf, a, b, b)
} by {
    if a < b {
        taylor3_aux_at_a_zero(f, df, ddf, a, b)
        taylor3_aux(f, df, ddf, a, b, a) = Real.0
        taylor3_aux_at_b_zero(f, df, ddf, a, b)
        taylor3_aux(f, df, ddf, a, b, b) = Real.0
        taylor3_aux(f, df, ddf, a, b, a) = taylor3_aux(f, df, ddf, a, b, b)
    }
}

/// The auxiliary function of the third-order Taylor theorem is continuous on
/// every closed interval when f, df and ddf are differentiable everywhere.
theorem taylor3_aux_continuous_on_closed(f: Real -> Real, df: Real -> Real, ddf: Real -> Real, dddf: Real -> Real, a: Real, b: Real, u: Real, v: Real) {
    is_derivative_fn(f, df) and is_derivative_fn(df, ddf) and is_derivative_fn(ddf, dddf)
    implies continuous_on_closed(taylor3_aux(f, df, ddf, a, b), u, v)
} by {
    if is_derivative_fn(f, df) and is_derivative_fn(df, ddf) and is_derivative_fn(ddf, dddf) {
        forall(x: Real) {
            if closed_interval_set(u, v).contains(x) {
                is_derivative_fn_imp_continuous_at(f, df, x)
                continuous_at(f, x)
                is_derivative_fn_imp_continuous_at(df, ddf, x)
                continuous_at(df, x)
                is_derivative_fn_imp_continuous_at(ddf, dddf, x)
                continuous_at(ddf, x)
                constant_function_is_continuous_at(taylor_two * f(b), x)
                continuous_at(constant[Real, Real](taylor_two * f(b)), x)
                constant_function_is_continuous_at(taylor_two, x)
                continuous_at(constant[Real, Real](taylor_two), x)
                continuous_at_pointwise_mul(constant[Real, Real](taylor_two), f, x)
                continuous_at(pointwise_mul(constant[Real, Real](taylor_two), f), x)
                continuous_at_pointwise_neg(pointwise_mul(constant[Real, Real](taylor_two), f), x)
                continuous_at(pointwise_neg(pointwise_mul(constant[Real, Real](taylor_two), f)), x)
                continuous_at_pointwise_add(constant[Real, Real](taylor_two * f(b)),
                    pointwise_neg(pointwise_mul(constant[Real, Real](taylor_two), f)), x)
                continuous_at(pointwise_add(constant[Real, Real](taylor_two * f(b)),
                    pointwise_neg(pointwise_mul(constant[Real, Real](taylor_two), f))), x)
                continuous_at_affine_real(-Real.1, b, x)
                continuous_at(affine_real(-Real.1, b), x)
                continuous_at_pointwise_mul(pointwise_mul(constant[Real, Real](taylor_two), df),
                    affine_real(-Real.1, b), x)
                continuous_at(pointwise_mul(pointwise_mul(constant[Real, Real](taylor_two), df),
                    affine_real(-Real.1, b)), x)
                continuous_at_pointwise_neg(pointwise_mul(pointwise_mul(constant[Real, Real](taylor_two), df),
                    affine_real(-Real.1, b)), x)
                continuous_at(pointwise_neg(pointwise_mul(pointwise_mul(constant[Real, Real](taylor_two), df),
                    affine_real(-Real.1, b))), x)
                continuous_at_pointwise_add(pointwise_add(constant[Real, Real](taylor_two * f(b)),
                        pointwise_neg(pointwise_mul(constant[Real, Real](taylor_two), f))),
                    pointwise_neg(pointwise_mul(pointwise_mul(constant[Real, Real](taylor_two), df),
                        affine_real(-Real.1, b))), x)
                continuous_at(pointwise_add(pointwise_add(constant[Real, Real](taylor_two * f(b)),
                        pointwise_neg(pointwise_mul(constant[Real, Real](taylor_two), f))),
                    pointwise_neg(pointwise_mul(pointwise_mul(constant[Real, Real](taylor_two), df),
                        affine_real(-Real.1, b)))), x)
                continuous_at_square_real(affine_real(-Real.1, b, x))
                continuous_at(square_real, affine_real(-Real.1, b, x))
                continuous_at_compose(square_real, affine_real(-Real.1, b), x)
                continuous_at(compose(square_real, affine_real(-Real.1, b)), x)
                continuous_at_pointwise_mul(ddf, compose(square_real, affine_real(-Real.1, b)), x)
                continuous_at(pointwise_mul(ddf, compose(square_real, affine_real(-Real.1, b))), x)
                continuous_at_pointwise_neg(pointwise_mul(ddf, compose(square_real, affine_real(-Real.1, b))), x)
                continuous_at(pointwise_neg(pointwise_mul(ddf, compose(square_real, affine_real(-Real.1, b)))), x)
                continuous_at_pointwise_add(pointwise_add(pointwise_add(constant[Real, Real](taylor_two * f(b)),
                            pointwise_neg(pointwise_mul(constant[Real, Real](taylor_two), f))),
                        pointwise_neg(pointwise_mul(pointwise_mul(constant[Real, Real](taylor_two), df),
                            affine_real(-Real.1, b)))),
                    pointwise_neg(pointwise_mul(ddf, compose(square_real, affine_real(-Real.1, b)))), x)
                continuous_at(pointwise_add(pointwise_add(pointwise_add(constant[Real, Real](taylor_two * f(b)),
                            pointwise_neg(pointwise_mul(constant[Real, Real](taylor_two), f))),
                        pointwise_neg(pointwise_mul(pointwise_mul(constant[Real, Real](taylor_two), df),
                            affine_real(-Real.1, b)))),
                    pointwise_neg(pointwise_mul(ddf, compose(square_real, affine_real(-Real.1, b))))), x)
                continuous_at_cube_real(affine_real(-Real.1, b, x))
                continuous_at(cube_real, affine_real(-Real.1, b, x))
                continuous_at_compose(cube_real, affine_real(-Real.1, b), x)
                continuous_at(compose(cube_real, affine_real(-Real.1, b)), x)
                constant_function_is_continuous_at(taylor3_remainder_coeff(f, df, ddf, a, b), x)
                continuous_at(constant[Real, Real](taylor3_remainder_coeff(f, df, ddf, a, b)), x)
                continuous_at_pointwise_mul(constant[Real, Real](taylor3_remainder_coeff(f, df, ddf, a, b)),
                    compose(cube_real, affine_real(-Real.1, b)), x)
                continuous_at(pointwise_mul(constant[Real, Real](taylor3_remainder_coeff(f, df, ddf, a, b)),
                    compose(cube_real, affine_real(-Real.1, b))), x)
                continuous_at_pointwise_neg(pointwise_mul(constant[Real, Real](taylor3_remainder_coeff(f, df, ddf, a, b)),
                    compose(cube_real, affine_real(-Real.1, b))), x)
                continuous_at(pointwise_neg(pointwise_mul(constant[Real, Real](taylor3_remainder_coeff(f, df, ddf, a, b)),
                    compose(cube_real, affine_real(-Real.1, b)))), x)
                continuous_at_pointwise_add(pointwise_add(pointwise_add(pointwise_add(constant[Real, Real](taylor_two * f(b)),
                                pointwise_neg(pointwise_mul(constant[Real, Real](taylor_two), f))),
                            pointwise_neg(pointwise_mul(pointwise_mul(constant[Real, Real](taylor_two), df),
                                affine_real(-Real.1, b)))),
                        pointwise_neg(pointwise_mul(ddf, compose(square_real, affine_real(-Real.1, b))))),
                    pointwise_neg(pointwise_mul(constant[Real, Real](taylor3_remainder_coeff(f, df, ddf, a, b)),
                        compose(cube_real, affine_real(-Real.1, b)))), x)
                continuous_at(pointwise_add(pointwise_add(pointwise_add(pointwise_add(constant[Real, Real](taylor_two * f(b)),
                                pointwise_neg(pointwise_mul(constant[Real, Real](taylor_two), f))),
                            pointwise_neg(pointwise_mul(pointwise_mul(constant[Real, Real](taylor_two), df),
                                affine_real(-Real.1, b)))),
                        pointwise_neg(pointwise_mul(ddf, compose(square_real, affine_real(-Real.1, b))))),
                    pointwise_neg(pointwise_mul(constant[Real, Real](taylor3_remainder_coeff(f, df, ddf, a, b)),
                        compose(cube_real, affine_real(-Real.1, b))))), x)
                taylor3_aux_eq_pointwise(f, df, ddf, a, b)
                taylor3_aux(f, df, ddf, a, b) = taylor3_aux_pointwise(f, df, ddf, a, b)
                function_eq_transport_predicate_rev(
                    function(h: Real -> Real) { continuous_at(h, x) },
                    taylor3_aux(f, df, ddf, a, b),
                    taylor3_aux_pointwise(f, df, ddf, a, b))
                continuous_at(taylor3_aux(f, df, ddf, a, b), x)
            }
        }
        continuous_on_closed(taylor3_aux(f, df, ddf, a, b), u, v) = forall(x: Real) {
            closed_interval_set(u, v).contains(x) implies
                continuous_at(taylor3_aux(f, df, ddf, a, b), x)
        }
        continuous_on_closed(taylor3_aux(f, df, ddf, a, b), u, v)
    }
}

/// The square of the shift x ↦ (b - x)^2 has derivative -2 (b - x) at every point.
theorem taylor3_sq_has_derivative_at(b: Real, x0: Real) {
    has_derivative_at(compose(square_real, affine_real(-Real.1, b)), x0,
        -taylor_two * (b - x0))
} by {
    let affine = pointwise_add(pointwise_mul(constant[Real, Real](-Real.1), identity_fn[Real]), constant[Real, Real](b))
    derivative_square_real_after_affine(-Real.1, b, x0)
    has_derivative_at(compose(square_real, affine), x0,
        (affine(x0) * Real.1 + affine(x0) * Real.1) * -Real.1)
    affine_real_eq_pointwise_affine(-Real.1, b)
    affine_real(-Real.1, b) = affine
    function_eq_transport_predicate_rev(
        function(h: Real -> Real) {
            has_derivative_at(compose(square_real, h), x0,
                (affine(x0) * Real.1 + affine(x0) * Real.1) * -Real.1)
        },
        affine_real(-Real.1, b), affine)
    has_derivative_at(compose(square_real, affine_real(-Real.1, b)), x0,
        (affine(x0) * Real.1 + affine(x0) * Real.1) * -Real.1)
    affine(x0) = -Real.1 * x0 + b
    mul_neg_left(Real.1, x0)
    -Real.1 * x0 = -(Real.1 * x0)
    mul_one_left(x0)
    Real.1 * x0 = x0
    -(Real.1 * x0) = -x0
    -Real.1 * x0 = -x0
    -x0 + b = b - x0
    affine(x0) = b - x0
    mul_one_right(affine(x0))
    affine(x0) * Real.1 = affine(x0)
    affine(x0) * Real.1 + affine(x0) * Real.1 = affine(x0) + affine(x0)
    mul_one_left(affine(x0))
    Real.1 * affine(x0) = affine(x0)
    mul_distrib_left(Real.1, Real.1, affine(x0))
    (Real.1 + Real.1) * affine(x0) = Real.1 * affine(x0) + Real.1 * affine(x0)
    (Real.1 + Real.1) * affine(x0) = affine(x0) + affine(x0)
    taylor_two = Real.1 + Real.1
    taylor_two * affine(x0) = (Real.1 + Real.1) * affine(x0)
    taylor_two * affine(x0) = affine(x0) + affine(x0)
    affine(x0) * Real.1 + affine(x0) * Real.1 = taylor_two * affine(x0)
    (affine(x0) * Real.1 + affine(x0) * Real.1) * -Real.1 =
        taylor_two * affine(x0) * -Real.1
    mul_neg_right(taylor_two * affine(x0), Real.1)
    (taylor_two * affine(x0)) * -Real.1 = -(taylor_two * affine(x0))
    taylor_two * affine(x0) * -Real.1 = -(taylor_two * affine(x0))
    (affine(x0) * Real.1 + affine(x0) * Real.1) * -Real.1 =
        -(taylor_two * affine(x0))
    taylor_two * affine(x0) = taylor_two * (b - x0)
    -(taylor_two * affine(x0)) = -(taylor_two * (b - x0))
    (affine(x0) * Real.1 + affine(x0) * Real.1) * -Real.1 =
        -(taylor_two * (b - x0))
    -taylor_two * (b - x0) = -(taylor_two * (b - x0))
    (affine(x0) * Real.1 + affine(x0) * Real.1) * -Real.1 =
        -taylor_two * (b - x0)
    has_derivative_at(compose(square_real, affine_real(-Real.1, b)), x0,
        -taylor_two * (b - x0))
}

/// The cube of the shift x ↦ (b - x)^3 has derivative -3 (b - x)^2 at every point.
theorem taylor3_cube_has_derivative_at(b: Real, x0: Real) {
    has_derivative_at(compose(cube_real, affine_real(-Real.1, b)), x0,
        -taylor_three * (b - x0).pow(Nat.2))
} by {
    let affine = pointwise_add(pointwise_mul(constant[Real, Real](-Real.1), identity_fn[Real]), constant[Real, Real](b))
    derivative_cube_real_after_affine(-Real.1, b, x0)
    has_derivative_at(compose(cube_real, affine), x0,
        (square_real(affine(x0)) * Real.1 +
            affine(x0) * (affine(x0) * Real.1 + affine(x0) * Real.1)) * -Real.1)
    affine_real_eq_pointwise_affine(-Real.1, b)
    affine_real(-Real.1, b) = affine
    function_eq_transport_predicate_rev(
        function(h: Real -> Real) {
            has_derivative_at(compose(cube_real, h), x0,
                (square_real(affine(x0)) * Real.1 +
                    affine(x0) * (affine(x0) * Real.1 + affine(x0) * Real.1)) * -Real.1)
        },
        affine_real(-Real.1, b), affine)
    has_derivative_at(compose(cube_real, affine_real(-Real.1, b)), x0,
        (square_real(affine(x0)) * Real.1 +
            affine(x0) * (affine(x0) * Real.1 + affine(x0) * Real.1)) * -Real.1)
    affine(x0) = -Real.1 * x0 + b
    mul_neg_left(Real.1, x0)
    -Real.1 * x0 = -(Real.1 * x0)
    mul_one_left(x0)
    Real.1 * x0 = x0
    -(Real.1 * x0) = -x0
    -Real.1 * x0 = -x0
    -x0 + b = b - x0
    affine(x0) = b - x0
    real_pow_two_eq_mul(b - x0)
    (b - x0).pow(Nat.2) = (b - x0) * (b - x0)
    square_real(affine(x0)) = square_real(b - x0)
    square_real(b - x0) = (b - x0) * (b - x0)
    square_real(affine(x0)) = (b - x0).pow(Nat.2)
    mul_one_right(affine(x0))
    affine(x0) * Real.1 = affine(x0)
    mul_one_left(affine(x0))
    Real.1 * affine(x0) = affine(x0)
    mul_distrib_left(Real.1, Real.1, affine(x0))
    (Real.1 + Real.1) * affine(x0) = Real.1 * affine(x0) + Real.1 * affine(x0)
    (Real.1 + Real.1) * affine(x0) = affine(x0) + affine(x0)
    taylor_two = Real.1 + Real.1
    taylor_two * affine(x0) = (Real.1 + Real.1) * affine(x0)
    taylor_two * affine(x0) = affine(x0) + affine(x0)
    affine(x0) * Real.1 + affine(x0) * Real.1 = taylor_two * affine(x0)
    square_real(affine(x0)) * Real.1 +
        affine(x0) * (affine(x0) * Real.1 + affine(x0) * Real.1) =
        (b - x0).pow(Nat.2) + affine(x0) * (taylor_two * affine(x0))
    mul_assoc(affine(x0), taylor_two, affine(x0))
    affine(x0) * (taylor_two * affine(x0)) =
        (affine(x0) * taylor_two) * affine(x0)
    real_mul_comm(affine(x0), taylor_two)
    affine(x0) * taylor_two = taylor_two * affine(x0)
    affine(x0) * (taylor_two * affine(x0)) =
        (taylor_two * affine(x0)) * affine(x0)
    real_mul_comm(taylor_two, affine(x0))
    taylor_two * affine(x0) = affine(x0) * taylor_two
    mul_assoc(affine(x0), affine(x0), taylor_two)
    (affine(x0) * affine(x0)) * taylor_two =
        affine(x0) * (affine(x0) * taylor_two)
    affine(x0) * (taylor_two * affine(x0)) =
        (affine(x0) * affine(x0)) * taylor_two
    square_real(affine(x0)) = affine(x0) * affine(x0)
    (affine(x0) * affine(x0)) * taylor_two = square_real(affine(x0)) * taylor_two
    affine(x0) * (taylor_two * affine(x0)) = square_real(affine(x0)) * taylor_two
    (b - x0).pow(Nat.2) + affine(x0) * (taylor_two * affine(x0)) =
        (b - x0).pow(Nat.2) + square_real(affine(x0)) * taylor_two
    square_real(affine(x0)) * taylor_two = (b - x0).pow(Nat.2) * taylor_two
    (b - x0).pow(Nat.2) + affine(x0) * (taylor_two * affine(x0)) =
        (b - x0).pow(Nat.2) + (b - x0).pow(Nat.2) * taylor_two
    mul_distrib_right((b - x0).pow(Nat.2), Real.1, taylor_two)
    (b - x0).pow(Nat.2) * (Real.1 + taylor_two) =
        (b - x0).pow(Nat.2) * Real.1 + (b - x0).pow(Nat.2) * taylor_two
    mul_one_right((b - x0).pow(Nat.2))
    (b - x0).pow(Nat.2) * Real.1 = (b - x0).pow(Nat.2)
    (b - x0).pow(Nat.2) * (Real.1 + taylor_two) =
        (b - x0).pow(Nat.2) + (b - x0).pow(Nat.2) * taylor_two
    Real.1 + taylor_two = taylor_three
    (b - x0).pow(Nat.2) * (Real.1 + taylor_two) =
        (b - x0).pow(Nat.2) * taylor_three
    (b - x0).pow(Nat.2) + (b - x0).pow(Nat.2) * taylor_two =
        (b - x0).pow(Nat.2) * taylor_three
    square_real(affine(x0)) * Real.1 +
        affine(x0) * (affine(x0) * Real.1 + affine(x0) * Real.1) =
        (b - x0).pow(Nat.2) * taylor_three
    mul_neg_right((b - x0).pow(Nat.2) * taylor_three, Real.1)
    ((b - x0).pow(Nat.2) * taylor_three) * -Real.1 =
        -((b - x0).pow(Nat.2) * taylor_three)
    (square_real(affine(x0)) * Real.1 +
        affine(x0) * (affine(x0) * Real.1 + affine(x0) * Real.1)) * -Real.1 =
        ((b - x0).pow(Nat.2) * taylor_three) * -Real.1
    (square_real(affine(x0)) * Real.1 +
        affine(x0) * (affine(x0) * Real.1 + affine(x0) * Real.1)) * -Real.1 =
        -((b - x0).pow(Nat.2) * taylor_three)
    mul_assoc((b - x0).pow(Nat.2), taylor_three, Real.1)
    ((b - x0).pow(Nat.2) * taylor_three) * Real.1 =
        (b - x0).pow(Nat.2) * (taylor_three * Real.1)
    mul_one_right(taylor_three)
    taylor_three * Real.1 = taylor_three
    ((b - x0).pow(Nat.2) * taylor_three) * Real.1 =
        (b - x0).pow(Nat.2) * taylor_three
    real_mul_comm((b - x0).pow(Nat.2), taylor_three)
    (b - x0).pow(Nat.2) * taylor_three = taylor_three * (b - x0).pow(Nat.2)
    -((b - x0).pow(Nat.2) * taylor_three) =
        -(taylor_three * (b - x0).pow(Nat.2))
    (square_real(affine(x0)) * Real.1 +
        affine(x0) * (affine(x0) * Real.1 + affine(x0) * Real.1)) * -Real.1 =
        -(taylor_three * (b - x0).pow(Nat.2))
    -taylor_three * (b - x0).pow(Nat.2) =
        -(taylor_three * (b - x0).pow(Nat.2))
    (square_real(affine(x0)) * Real.1 +
        affine(x0) * (affine(x0) * Real.1 + affine(x0) * Real.1)) * -Real.1 =
        -taylor_three * (b - x0).pow(Nat.2)
    has_derivative_at(compose(cube_real, affine_real(-Real.1, b)), x0,
        -taylor_three * (b - x0).pow(Nat.2))
}

/// The auxiliary function of the third-order Taylor theorem has the expected
/// global derivative function.
theorem taylor3_aux_is_derivative_fn(f: Real -> Real, df: Real -> Real, ddf: Real -> Real, dddf: Real -> Real, a: Real, b: Real) {
    is_derivative_fn(f, df) and is_derivative_fn(df, ddf) and is_derivative_fn(ddf, dddf)
    implies is_derivative_fn(taylor3_aux(f, df, ddf, a, b),
        taylor3_aux_derivative(f, df, ddf, dddf, a, b))
} by {
    if is_derivative_fn(f, df) and is_derivative_fn(df, ddf) and is_derivative_fn(ddf, dddf) {
        forall(x: Real) {
            is_derivative_fn_at(f, df, x)
            has_derivative_at(f, x, df(x))
            derivative_pointwise_const_mul(taylor_two, f, x, df(x))
            has_derivative_at(pointwise_mul(constant[Real, Real](taylor_two), f), x,
                taylor_two * df(x))
            derivative_pointwise_neg(pointwise_mul(constant[Real, Real](taylor_two), f),
                x, taylor_two * df(x))
            has_derivative_at(pointwise_neg(pointwise_mul(constant[Real, Real](taylor_two), f)),
                x, -(taylor_two * df(x)))
            constant_has_derivative_at(taylor_two * f(b), x)
            has_derivative_at(constant[Real, Real](taylor_two * f(b)), x, Real.0)
            derivative_pointwise_add(constant[Real, Real](taylor_two * f(b)),
                pointwise_neg(pointwise_mul(constant[Real, Real](taylor_two), f)),
                x, Real.0, -(taylor_two * df(x)))
            has_derivative_at(pointwise_add(constant[Real, Real](taylor_two * f(b)),
                pointwise_neg(pointwise_mul(constant[Real, Real](taylor_two), f))),
                x, Real.0 + -(taylor_two * df(x)))
            affine_real_has_derivative_at(-Real.1, b, x)
            has_derivative_at(affine_real(-Real.1, b), x, -Real.1)
            is_derivative_fn_at(df, ddf, x)
            has_derivative_at(df, x, ddf(x))
            derivative_pointwise_const_mul(taylor_two, df, x, ddf(x))
            has_derivative_at(pointwise_mul(constant[Real, Real](taylor_two), df), x,
                taylor_two * ddf(x))
            derivative_pointwise_mul(pointwise_mul(constant[Real, Real](taylor_two), df),
                affine_real(-Real.1, b), x, taylor_two * ddf(x), -Real.1)
            has_derivative_at(pointwise_mul(pointwise_mul(constant[Real, Real](taylor_two), df),
                affine_real(-Real.1, b)), x,
                pointwise_mul(constant[Real, Real](taylor_two), df, x) * -Real.1 +
                    affine_real(-Real.1, b, x) * (taylor_two * ddf(x)))
            derivative_pointwise_neg(pointwise_mul(pointwise_mul(constant[Real, Real](taylor_two), df),
                affine_real(-Real.1, b)), x,
                pointwise_mul(constant[Real, Real](taylor_two), df, x) * -Real.1 +
                    affine_real(-Real.1, b, x) * (taylor_two * ddf(x)))
            has_derivative_at(pointwise_neg(pointwise_mul(pointwise_mul(constant[Real, Real](taylor_two), df),
                affine_real(-Real.1, b))), x,
                -(pointwise_mul(constant[Real, Real](taylor_two), df, x) * -Real.1 +
                    affine_real(-Real.1, b, x) * (taylor_two * ddf(x))))
            derivative_pointwise_add(pointwise_add(constant[Real, Real](taylor_two * f(b)),
                    pointwise_neg(pointwise_mul(constant[Real, Real](taylor_two), f))),
                pointwise_neg(pointwise_mul(pointwise_mul(constant[Real, Real](taylor_two), df),
                    affine_real(-Real.1, b))),
                x, Real.0 + -(taylor_two * df(x)),
                -(pointwise_mul(constant[Real, Real](taylor_two), df, x) * -Real.1 +
                    affine_real(-Real.1, b, x) * (taylor_two * ddf(x))))
            has_derivative_at(pointwise_add(pointwise_add(constant[Real, Real](taylor_two * f(b)),
                    pointwise_neg(pointwise_mul(constant[Real, Real](taylor_two), f))),
                pointwise_neg(pointwise_mul(pointwise_mul(constant[Real, Real](taylor_two), df),
                    affine_real(-Real.1, b)))), x,
                (Real.0 + -(taylor_two * df(x))) +
                    -(pointwise_mul(constant[Real, Real](taylor_two), df, x) * -Real.1 +
                        affine_real(-Real.1, b, x) * (taylor_two * ddf(x))))
            taylor3_sq_has_derivative_at(b, x)
            has_derivative_at(compose(square_real, affine_real(-Real.1, b)), x,
                -taylor_two * (b - x))
            is_derivative_fn_at(ddf, dddf, x)
            has_derivative_at(ddf, x, dddf(x))
            derivative_pointwise_mul(ddf, compose(square_real, affine_real(-Real.1, b)),
                x, dddf(x), -taylor_two * (b - x))
            has_derivative_at(pointwise_mul(ddf, compose(square_real, affine_real(-Real.1, b))),
                x, ddf(x) * (-taylor_two * (b - x)) +
                    compose(square_real, affine_real(-Real.1, b), x) * dddf(x))
            derivative_pointwise_neg(pointwise_mul(ddf, compose(square_real, affine_real(-Real.1, b))),
                x, ddf(x) * (-taylor_two * (b - x)) +
                    compose(square_real, affine_real(-Real.1, b), x) * dddf(x))
            has_derivative_at(pointwise_neg(pointwise_mul(ddf, compose(square_real, affine_real(-Real.1, b)))),
                x, -(ddf(x) * (-taylor_two * (b - x)) +
                    compose(square_real, affine_real(-Real.1, b), x) * dddf(x)))
            derivative_pointwise_add(pointwise_add(pointwise_add(constant[Real, Real](taylor_two * f(b)),
                        pointwise_neg(pointwise_mul(constant[Real, Real](taylor_two), f))),
                    pointwise_neg(pointwise_mul(pointwise_mul(constant[Real, Real](taylor_two), df),
                        affine_real(-Real.1, b)))),
                pointwise_neg(pointwise_mul(ddf, compose(square_real, affine_real(-Real.1, b)))),
                x,
                (Real.0 + -(taylor_two * df(x))) +
                    -(pointwise_mul(constant[Real, Real](taylor_two), df, x) * -Real.1 +
                        affine_real(-Real.1, b, x) * (taylor_two * ddf(x))),
                -(ddf(x) * (-taylor_two * (b - x)) +
                    compose(square_real, affine_real(-Real.1, b), x) * dddf(x)))
            has_derivative_at(pointwise_add(pointwise_add(pointwise_add(constant[Real, Real](taylor_two * f(b)),
                            pointwise_neg(pointwise_mul(constant[Real, Real](taylor_two), f))),
                        pointwise_neg(pointwise_mul(pointwise_mul(constant[Real, Real](taylor_two), df),
                            affine_real(-Real.1, b)))),
                    pointwise_neg(pointwise_mul(ddf, compose(square_real, affine_real(-Real.1, b))))),
                x,
                ((Real.0 + -(taylor_two * df(x))) +
                    -(pointwise_mul(constant[Real, Real](taylor_two), df, x) * -Real.1 +
                        affine_real(-Real.1, b, x) * (taylor_two * ddf(x)))) +
                    -(ddf(x) * (-taylor_two * (b - x)) +
                        compose(square_real, affine_real(-Real.1, b), x) * dddf(x)))
            taylor3_cube_has_derivative_at(b, x)
            has_derivative_at(compose(cube_real, affine_real(-Real.1, b)), x,
                -taylor_three * (b - x).pow(Nat.2))
            derivative_pointwise_const_mul(taylor3_remainder_coeff(f, df, ddf, a, b),
                compose(cube_real, affine_real(-Real.1, b)), x,
                -taylor_three * (b - x).pow(Nat.2))
            has_derivative_at(pointwise_mul(constant[Real, Real](taylor3_remainder_coeff(f, df, ddf, a, b)),
                compose(cube_real, affine_real(-Real.1, b))), x,
                taylor3_remainder_coeff(f, df, ddf, a, b) * (-taylor_three * (b - x).pow(Nat.2)))
            derivative_pointwise_neg(pointwise_mul(constant[Real, Real](taylor3_remainder_coeff(f, df, ddf, a, b)),
                compose(cube_real, affine_real(-Real.1, b))), x,
                taylor3_remainder_coeff(f, df, ddf, a, b) * (-taylor_three * (b - x).pow(Nat.2)))
            has_derivative_at(pointwise_neg(pointwise_mul(constant[Real, Real](taylor3_remainder_coeff(f, df, ddf, a, b)),
                compose(cube_real, affine_real(-Real.1, b)))), x,
                -(taylor3_remainder_coeff(f, df, ddf, a, b) * (-taylor_three * (b - x).pow(Nat.2))))
            derivative_pointwise_add(pointwise_add(pointwise_add(pointwise_add(constant[Real, Real](taylor_two * f(b)),
                            pointwise_neg(pointwise_mul(constant[Real, Real](taylor_two), f))),
                        pointwise_neg(pointwise_mul(pointwise_mul(constant[Real, Real](taylor_two), df),
                            affine_real(-Real.1, b)))),
                    pointwise_neg(pointwise_mul(ddf, compose(square_real, affine_real(-Real.1, b))))),
                pointwise_neg(pointwise_mul(constant[Real, Real](taylor3_remainder_coeff(f, df, ddf, a, b)),
                    compose(cube_real, affine_real(-Real.1, b)))),
                x,
                ((Real.0 + -(taylor_two * df(x))) +
                    -(pointwise_mul(constant[Real, Real](taylor_two), df, x) * -Real.1 +
                        affine_real(-Real.1, b, x) * (taylor_two * ddf(x)))) +
                    -(ddf(x) * (-taylor_two * (b - x)) +
                        compose(square_real, affine_real(-Real.1, b), x) * dddf(x)),
                -(taylor3_remainder_coeff(f, df, ddf, a, b) * (-taylor_three * (b - x).pow(Nat.2))))
            has_derivative_at(pointwise_add(pointwise_add(pointwise_add(pointwise_add(constant[Real, Real](taylor_two * f(b)),
                                pointwise_neg(pointwise_mul(constant[Real, Real](taylor_two), f))),
                            pointwise_neg(pointwise_mul(pointwise_mul(constant[Real, Real](taylor_two), df),
                                affine_real(-Real.1, b)))),
                        pointwise_neg(pointwise_mul(ddf, compose(square_real, affine_real(-Real.1, b))))),
                    pointwise_neg(pointwise_mul(constant[Real, Real](taylor3_remainder_coeff(f, df, ddf, a, b)),
                        compose(cube_real, affine_real(-Real.1, b))))), x,
                (((Real.0 + -(taylor_two * df(x))) +
                    -(pointwise_mul(constant[Real, Real](taylor_two), df, x) * -Real.1 +
                        affine_real(-Real.1, b, x) * (taylor_two * ddf(x)))) +
                    -(ddf(x) * (-taylor_two * (b - x)) +
                        compose(square_real, affine_real(-Real.1, b), x) * dddf(x))) +
                    -(taylor3_remainder_coeff(f, df, ddf, a, b) * (-taylor_three * (b - x).pow(Nat.2))))
            // Now simplify the rule-produced derivative value to the simplified form.
            pointwise_mul(constant[Real, Real](taylor_two), df, x) = taylor_two * df(x)
            pointwise_mul(constant[Real, Real](taylor_two), df, x) * -Real.1 =
                (taylor_two * df(x)) * -Real.1
            mul_neg_right(taylor_two * df(x), Real.1)
            (taylor_two * df(x)) * -Real.1 = -(taylor_two * df(x))
            pointwise_mul(constant[Real, Real](taylor_two), df, x) * -Real.1 =
                -(taylor_two * df(x))
            affine_real(-Real.1, b, x) = -Real.1 * x + b
            mul_neg_left(Real.1, x)
            -Real.1 * x = -(Real.1 * x)
            mul_one_left(x)
            Real.1 * x = x
            -(Real.1 * x) = -x
            -Real.1 * x = -x
            -x + b = b - x
            affine_real(-Real.1, b, x) = b - x
            pointwise_mul(constant[Real, Real](taylor_two), df, x) * -Real.1 +
                affine_real(-Real.1, b, x) * (taylor_two * ddf(x)) =
                -(taylor_two * df(x)) + (b - x) * (taylor_two * ddf(x))
            Real.0 + -(taylor_two * df(x)) = -(taylor_two * df(x))
            compose(square_real, affine_real(-Real.1, b), x) =
                square_real(affine_real(-Real.1, b, x))
            square_real(affine_real(-Real.1, b, x)) = square_real(b - x)
            real_pow_two_eq_mul(b - x)
            (b - x).pow(Nat.2) = (b - x) * (b - x)
            square_real(b - x) = (b - x).pow(Nat.2)
            compose(square_real, affine_real(-Real.1, b), x) = (b - x).pow(Nat.2)
            // The product rule value for 2df * g1, negated, distributes.
            -(pointwise_mul(constant[Real, Real](taylor_two), df, x) * -Real.1 +
                affine_real(-Real.1, b, x) * (taylor_two * ddf(x))) =
                -(-(taylor_two * df(x)) + (b - x) * (taylor_two * ddf(x)))
            neg_distrib(-(taylor_two * df(x)), (b - x) * (taylor_two * ddf(x)))
            -(-(taylor_two * df(x)) + (b - x) * (taylor_two * ddf(x))) =
                -(-(taylor_two * df(x))) + -((b - x) * (taylor_two * ddf(x)))
            neg_neg(taylor_two * df(x))
            -(-(taylor_two * df(x))) = taylor_two * df(x)
            -(-(taylor_two * df(x))) + -((b - x) * (taylor_two * ddf(x))) =
                taylor_two * df(x) + -((b - x) * (taylor_two * ddf(x)))
            -(-(taylor_two * df(x)) + (b - x) * (taylor_two * ddf(x))) =
                taylor_two * df(x) + -((b - x) * (taylor_two * ddf(x)))
            -(pointwise_mul(constant[Real, Real](taylor_two), df, x) * -Real.1 +
                affine_real(-Real.1, b, x) * (taylor_two * ddf(x))) =
                taylor_two * df(x) + -((b - x) * (taylor_two * ddf(x)))
            (Real.0 + -(taylor_two * df(x))) +
                -(pointwise_mul(constant[Real, Real](taylor_two), df, x) * -Real.1 +
                    affine_real(-Real.1, b, x) * (taylor_two * ddf(x))) =
                -(taylor_two * df(x)) + taylor_two * df(x) + -((b - x) * (taylor_two * ddf(x)))
            -(taylor_two * df(x)) + taylor_two * df(x) = Real.0
            -(taylor_two * df(x)) + taylor_two * df(x) + -((b - x) * (taylor_two * ddf(x))) =
                -((b - x) * (taylor_two * ddf(x)))
            (Real.0 + -(taylor_two * df(x))) +
                -(pointwise_mul(constant[Real, Real](taylor_two), df, x) * -Real.1 +
                    affine_real(-Real.1, b, x) * (taylor_two * ddf(x))) =
                -((b - x) * (taylor_two * ddf(x)))
            // The product rule value for ddf * g2, negated, distributes.
            mul_neg_right(ddf(x), taylor_two * (b - x))
            ddf(x) * (-(taylor_two * (b - x))) = -(ddf(x) * (taylor_two * (b - x)))
            ddf(x) * (-taylor_two * (b - x)) = ddf(x) * (-(taylor_two * (b - x)))
            ddf(x) * (-taylor_two * (b - x)) = -(ddf(x) * (taylor_two * (b - x)))
            neg_distrib(-(ddf(x) * (taylor_two * (b - x))), (b - x).pow(Nat.2) * dddf(x))
            -(-(ddf(x) * (taylor_two * (b - x))) + (b - x).pow(Nat.2) * dddf(x)) =
                -(-(ddf(x) * (taylor_two * (b - x)))) + -((b - x).pow(Nat.2) * dddf(x))
            neg_neg(ddf(x) * (taylor_two * (b - x)))
            -(-(ddf(x) * (taylor_two * (b - x)))) = ddf(x) * (taylor_two * (b - x))
            -(-(ddf(x) * (taylor_two * (b - x)))) + -((b - x).pow(Nat.2) * dddf(x)) =
                ddf(x) * (taylor_two * (b - x)) + -((b - x).pow(Nat.2) * dddf(x))
            -(ddf(x) * (-taylor_two * (b - x)) +
                compose(square_real, affine_real(-Real.1, b), x) * dddf(x)) =
                ddf(x) * (taylor_two * (b - x)) + -((b - x).pow(Nat.2) * dddf(x))
            // The two quadratic terms cancel.
            mul_assoc(ddf(x), taylor_two, b - x)
            ddf(x) * (taylor_two * (b - x)) = (ddf(x) * taylor_two) * (b - x)
            real_mul_comm(ddf(x), taylor_two)
            ddf(x) * taylor_two = taylor_two * ddf(x)
            (ddf(x) * taylor_two) * (b - x) = (taylor_two * ddf(x)) * (b - x)
            mul_assoc(taylor_two, ddf(x), b - x)
            (taylor_two * ddf(x)) * (b - x) = taylor_two * (ddf(x) * (b - x))
            mul_assoc(b - x, taylor_two, ddf(x))
            (b - x) * (taylor_two * ddf(x)) = ((b - x) * taylor_two) * ddf(x)
            real_mul_comm(b - x, taylor_two)
            (b - x) * taylor_two = taylor_two * (b - x)
            ((b - x) * taylor_two) * ddf(x) = (taylor_two * (b - x)) * ddf(x)
            mul_assoc(taylor_two, b - x, ddf(x))
            (taylor_two * (b - x)) * ddf(x) = taylor_two * ((b - x) * ddf(x))
            real_mul_comm(b - x, ddf(x))
            (b - x) * ddf(x) = ddf(x) * (b - x)
            taylor_two * ((b - x) * ddf(x)) = taylor_two * (ddf(x) * (b - x))
            (taylor_two * (b - x)) * ddf(x) = taylor_two * (ddf(x) * (b - x))
            (b - x) * (taylor_two * ddf(x)) = taylor_two * (ddf(x) * (b - x))
            ddf(x) * (taylor_two * (b - x)) = taylor_two * (ddf(x) * (b - x))
            ddf(x) * (taylor_two * (b - x)) = (b - x) * (taylor_two * ddf(x))
            add_neg_eq_zero((b - x) * (taylor_two * ddf(x)))
            (b - x) * (taylor_two * ddf(x)) + -((b - x) * (taylor_two * ddf(x))) = Real.0
            add_comm(-((b - x) * (taylor_two * ddf(x))),
                (b - x) * (taylor_two * ddf(x)))
            -((b - x) * (taylor_two * ddf(x))) + (b - x) * (taylor_two * ddf(x)) = Real.0
            -((b - x) * (taylor_two * ddf(x))) + ddf(x) * (taylor_two * (b - x)) = Real.0
            add_assoc(-((b - x) * (taylor_two * ddf(x))),
                ddf(x) * (taylor_two * (b - x)), -((b - x).pow(Nat.2) * dddf(x)))
            -((b - x) * (taylor_two * ddf(x))) +
                (ddf(x) * (taylor_two * (b - x)) + -((b - x).pow(Nat.2) * dddf(x))) =
                (-((b - x) * (taylor_two * ddf(x))) + ddf(x) * (taylor_two * (b - x))) +
                    -((b - x).pow(Nat.2) * dddf(x))
            (-((b - x) * (taylor_two * ddf(x))) + ddf(x) * (taylor_two * (b - x))) +
                -((b - x).pow(Nat.2) * dddf(x)) =
                Real.0 + -((b - x).pow(Nat.2) * dddf(x))
            add_zero_left(-((b - x).pow(Nat.2) * dddf(x)))
            Real.0 + -((b - x).pow(Nat.2) * dddf(x)) = -((b - x).pow(Nat.2) * dddf(x))
            -((b - x) * (taylor_two * ddf(x))) +
                (ddf(x) * (taylor_two * (b - x)) + -((b - x).pow(Nat.2) * dddf(x))) =
                -((b - x).pow(Nat.2) * dddf(x))
            ((Real.0 + -(taylor_two * df(x))) +
                -(pointwise_mul(constant[Real, Real](taylor_two), df, x) * -Real.1 +
                    affine_real(-Real.1, b, x) * (taylor_two * ddf(x)))) +
                -(ddf(x) * (-taylor_two * (b - x)) +
                    compose(square_real, affine_real(-Real.1, b), x) * dddf(x)) =
                -((b - x).pow(Nat.2) * dddf(x))
            // The cubic term's negation.
            mul_neg_right(taylor3_remainder_coeff(f, df, ddf, a, b), taylor_three * (b - x).pow(Nat.2))
            taylor3_remainder_coeff(f, df, ddf, a, b) * (-(taylor_three * (b - x).pow(Nat.2))) =
                -(taylor3_remainder_coeff(f, df, ddf, a, b) * (taylor_three * (b - x).pow(Nat.2)))
            taylor3_remainder_coeff(f, df, ddf, a, b) * (-taylor_three * (b - x).pow(Nat.2)) =
                taylor3_remainder_coeff(f, df, ddf, a, b) * (-(taylor_three * (b - x).pow(Nat.2)))
            taylor3_remainder_coeff(f, df, ddf, a, b) * (-taylor_three * (b - x).pow(Nat.2)) =
                -(taylor3_remainder_coeff(f, df, ddf, a, b) * (taylor_three * (b - x).pow(Nat.2)))
            neg_neg(taylor3_remainder_coeff(f, df, ddf, a, b) * (taylor_three * (b - x).pow(Nat.2)))
            -(-(taylor3_remainder_coeff(f, df, ddf, a, b) * (taylor_three * (b - x).pow(Nat.2)))) =
                taylor3_remainder_coeff(f, df, ddf, a, b) * (taylor_three * (b - x).pow(Nat.2))
            -(taylor3_remainder_coeff(f, df, ddf, a, b) * (-taylor_three * (b - x).pow(Nat.2))) =
                taylor3_remainder_coeff(f, df, ddf, a, b) * (taylor_three * (b - x).pow(Nat.2))
            // Assemble the full derivative and factor out (b - x)^2.
            (((Real.0 + -(taylor_two * df(x))) +
                -(pointwise_mul(constant[Real, Real](taylor_two), df, x) * -Real.1 +
                    affine_real(-Real.1, b, x) * (taylor_two * ddf(x)))) +
                -(ddf(x) * (-taylor_two * (b - x)) +
                    compose(square_real, affine_real(-Real.1, b), x) * dddf(x))) +
                -(taylor3_remainder_coeff(f, df, ddf, a, b) * (-taylor_three * (b - x).pow(Nat.2))) =
                -((b - x).pow(Nat.2) * dddf(x)) +
                    taylor3_remainder_coeff(f, df, ddf, a, b) * (taylor_three * (b - x).pow(Nat.2))
            mul_assoc(taylor3_remainder_coeff(f, df, ddf, a, b), taylor_three, (b - x).pow(Nat.2))
            taylor3_remainder_coeff(f, df, ddf, a, b) * (taylor_three * (b - x).pow(Nat.2)) =
                (taylor3_remainder_coeff(f, df, ddf, a, b) * taylor_three) * (b - x).pow(Nat.2)
            real_mul_comm(taylor3_remainder_coeff(f, df, ddf, a, b), taylor_three)
            taylor3_remainder_coeff(f, df, ddf, a, b) * taylor_three =
                taylor_three * taylor3_remainder_coeff(f, df, ddf, a, b)
            (taylor3_remainder_coeff(f, df, ddf, a, b) * taylor_three) * (b - x).pow(Nat.2) =
                (taylor_three * taylor3_remainder_coeff(f, df, ddf, a, b)) * (b - x).pow(Nat.2)
            taylor3_remainder_coeff(f, df, ddf, a, b) * (taylor_three * (b - x).pow(Nat.2)) =
                (taylor_three * taylor3_remainder_coeff(f, df, ddf, a, b)) * (b - x).pow(Nat.2)
            -((b - x).pow(Nat.2) * dddf(x)) +
                taylor3_remainder_coeff(f, df, ddf, a, b) * (taylor_three * (b - x).pow(Nat.2)) =
                -((b - x).pow(Nat.2) * dddf(x)) +
                    (taylor_three * taylor3_remainder_coeff(f, df, ddf, a, b)) * (b - x).pow(Nat.2)
            add_comm(-((b - x).pow(Nat.2) * dddf(x)),
                (taylor_three * taylor3_remainder_coeff(f, df, ddf, a, b)) * (b - x).pow(Nat.2))
            -((b - x).pow(Nat.2) * dddf(x)) +
                (taylor_three * taylor3_remainder_coeff(f, df, ddf, a, b)) * (b - x).pow(Nat.2) =
                (taylor_three * taylor3_remainder_coeff(f, df, ddf, a, b)) * (b - x).pow(Nat.2) -
                    (b - x).pow(Nat.2) * dddf(x)
            real_mul_comm((b - x).pow(Nat.2), taylor_three * taylor3_remainder_coeff(f, df, ddf, a, b))
            (taylor_three * taylor3_remainder_coeff(f, df, ddf, a, b)) * (b - x).pow(Nat.2) =
                (b - x).pow(Nat.2) * (taylor_three * taylor3_remainder_coeff(f, df, ddf, a, b))
            (taylor_three * taylor3_remainder_coeff(f, df, ddf, a, b)) * (b - x).pow(Nat.2) -
                (b - x).pow(Nat.2) * dddf(x) =
                (b - x).pow(Nat.2) * (taylor_three * taylor3_remainder_coeff(f, df, ddf, a, b)) -
                    (b - x).pow(Nat.2) * dddf(x)
            mul_distrib_right((b - x).pow(Nat.2),
                taylor_three * taylor3_remainder_coeff(f, df, ddf, a, b), -dddf(x))
            (b - x).pow(Nat.2) * ((taylor_three * taylor3_remainder_coeff(f, df, ddf, a, b)) + -dddf(x)) =
                (b - x).pow(Nat.2) * (taylor_three * taylor3_remainder_coeff(f, df, ddf, a, b)) +
                    (b - x).pow(Nat.2) * (-dddf(x))
            mul_neg_right((b - x).pow(Nat.2), dddf(x))
            (b - x).pow(Nat.2) * (-dddf(x)) = -((b - x).pow(Nat.2) * dddf(x))
            (b - x).pow(Nat.2) * ((taylor_three * taylor3_remainder_coeff(f, df, ddf, a, b)) + -dddf(x)) =
                (b - x).pow(Nat.2) * (taylor_three * taylor3_remainder_coeff(f, df, ddf, a, b)) -
                    (b - x).pow(Nat.2) * dddf(x)
            (b - x).pow(Nat.2) * (taylor_three * taylor3_remainder_coeff(f, df, ddf, a, b) - dddf(x)) =
                (b - x).pow(Nat.2) * (taylor_three * taylor3_remainder_coeff(f, df, ddf, a, b)) -
                    (b - x).pow(Nat.2) * dddf(x)
            (((Real.0 + -(taylor_two * df(x))) +
                -(pointwise_mul(constant[Real, Real](taylor_two), df, x) * -Real.1 +
                    affine_real(-Real.1, b, x) * (taylor_two * ddf(x)))) +
                -(ddf(x) * (-taylor_two * (b - x)) +
                    compose(square_real, affine_real(-Real.1, b), x) * dddf(x))) +
                -(taylor3_remainder_coeff(f, df, ddf, a, b) * (-taylor_three * (b - x).pow(Nat.2))) =
                (b - x).pow(Nat.2) * (taylor_three * taylor3_remainder_coeff(f, df, ddf, a, b) - dddf(x))
            taylor3_aux_derivative(f, df, ddf, dddf, a, b, x) =
                (b - x).pow(Nat.2) * (taylor_three * taylor3_remainder_coeff(f, df, ddf, a, b) - dddf(x))
            (((Real.0 + -(taylor_two * df(x))) +
                -(pointwise_mul(constant[Real, Real](taylor_two), df, x) * -Real.1 +
                    affine_real(-Real.1, b, x) * (taylor_two * ddf(x)))) +
                -(ddf(x) * (-taylor_two * (b - x)) +
                    compose(square_real, affine_real(-Real.1, b), x) * dddf(x))) +
                -(taylor3_remainder_coeff(f, df, ddf, a, b) * (-taylor_three * (b - x).pow(Nat.2))) =
                taylor3_aux_derivative(f, df, ddf, dddf, a, b, x)
            taylor3_aux_eq_pointwise(f, df, ddf, a, b)
            taylor3_aux(f, df, ddf, a, b) = taylor3_aux_pointwise(f, df, ddf, a, b)
            has_derivative_at(pointwise_add(pointwise_add(pointwise_add(pointwise_add(constant[Real, Real](taylor_two * f(b)),
                                    pointwise_neg(pointwise_mul(constant[Real, Real](taylor_two), f))),
                                pointwise_neg(pointwise_mul(pointwise_mul(constant[Real, Real](taylor_two), df),
                                    affine_real(-Real.1, b)))),
                            pointwise_neg(pointwise_mul(ddf, compose(square_real, affine_real(-Real.1, b))))),
                        pointwise_neg(pointwise_mul(constant[Real, Real](taylor3_remainder_coeff(f, df, ddf, a, b)),
                            compose(cube_real, affine_real(-Real.1, b))))), x,
                    taylor3_aux_derivative(f, df, ddf, dddf, a, b, x))
            has_derivative_at(taylor3_aux_pointwise(f, df, ddf, a, b), x,
                taylor3_aux_derivative(f, df, ddf, dddf, a, b, x))
            function_eq_transport_predicate_rev(
                function(h: Real -> Real) {
                    has_derivative_at(h, x, taylor3_aux_derivative(f, df, ddf, dddf, a, b, x))
                },
                taylor3_aux(f, df, ddf, a, b),
                taylor3_aux_pointwise(f, df, ddf, a, b))
            has_derivative_at(taylor3_aux(f, df, ddf, a, b), x,
                taylor3_aux_derivative(f, df, ddf, dddf, a, b, x))
        }
        is_derivative_fn_iff(taylor3_aux(f, df, ddf, a, b),
            taylor3_aux_derivative(f, df, ddf, dddf, a, b))
        is_derivative_fn(taylor3_aux(f, df, ddf, a, b),
            taylor3_aux_derivative(f, df, ddf, dddf, a, b))
    }
}

/// Subtraction distributes over multiplication on the right: (a - b) * c = a * c - b * c.
theorem taylor3_sub_mul(a: Real, b: Real, c: Real) {
    (a - b) * c = a * c - b * c
} by {
    a - b = a + -b
    (a - b) * c = (a + -b) * c
    mul_distrib_right(a, -b, c)
    (a + -b) * c = a * c + (-b) * c
    mul_neg_right(b, c)
    (-b) * c = -(b * c)
    a * c + (-b) * c = a * c + -(b * c)
    a * c - b * c = a * c + -(b * c)
    (a - b) * c = a * c - b * c
}

/// Subtracting a sum equals subtracting each term: a - (b + c) = a - b - c.
theorem taylor3_sub_sum(a: Real, b: Real, c: Real) {
    a - (b + c) = a - b - c
} by {
    a - (b + c) = a + -(b + c)
    neg_distrib(b, c)
    -(b + c) = -b + -c
    a + -(b + c) = a + (-b + -c)
    add_assoc(a, -b, -c)
    (a + -b) + -c = a + (-b + -c)
    a + -b = a - b
    (a - b) + -c = (a + -b) + -c
    a - b - c = (a - b) + -c
    (a - b) + -c = a + (-b + -c)
    a + (-b + -c) = a - b - c
    a - (b + c) = a - b - c
}

/// Adding the subtracted terms back: (a - b - c - d) + b + c + d = a.
theorem taylor3_sub_add_cancel3(a: Real, b: Real, c: Real, d: Real) {
    (a - b - c - d) + b + c + d = a
} by {
    taylor3_sub_sum(a - b, c, d)
    (a - b) - (c + d) = (a - b) - c - d
    a - b - c - d = (a - b) - c - d
    taylor2_sub_sub_add(a, b, c + d)
    (a - b - (c + d)) + b = a - (c + d)
    a - b - (c + d) = a - b - c - d
    (a - b - c - d) + b = a - (c + d)
    taylor3_sub_sum(a, c, d)
    a - (c + d) = a - c - d
    (a - b - c - d) + b = a - c - d
    taylor2_sub_sub_add(a, c, d)
    (a - c - d) + c = a - d
    ((a - b - c - d) + b) + c = a - d
    add_assoc(a - b - c - d, b, c)
    (a - b - c - d) + b + c = ((a - b - c - d) + b) + c
    (a - b - c - d) + b + c = a - d
    taylor2_sub_add_cancel(a, d)
    (a - d) + d = a
    ((a - b - c - d) + b + c) + d = (a - d) + d
    add_assoc(a - b - c - d, b, c + d)
    (a - b - c - d) + (b + c + d) = (a - b - c - d) + b + c + d
    add_assoc(b, c, d)
    (b + c) + d = b + c + d
    (a - b - c - d) + b + c + d = ((a - b - c - d) + b + c) + d
    (a - b - c - d) + b + c + d = a
}

/// A negation and its opposite cancel: -x + (x - y) = -y.
theorem taylor4_neg_cancel(x: Real, y: Real) {
    -x + (x - y) = -y
} by {
    x - y = x + -y
    -x + (x - y) = -x + (x + -y)
    add_assoc(-x, x, -y)
    -x + (x + -y) = (-x + x) + -y
    add_comm(x, -x)
    x + -x = -x + x
    add_neg_eq_zero(x)
    x + -x = Real.0
    -x + x = Real.0
    (-x + x) + -y = Real.0 + -y
    add_zero_left(-y)
    Real.0 + -y = -y
    -x + (x + -y) = -y
    -x + (x - y) = -y
}

/// Dividing a doubled difference chain by two recovers the difference chain.
theorem taylor3_div_two_chain(a: Real, b: Real, c: Real, d: Real) {
    (taylor_two * a - taylor_two * b - taylor_two * c - d) / taylor_two =
        a - b - c - d / taylor_two
} by {
    taylor_two_not_zero
    taylor_two != Real.0
    (taylor_two * a - taylor_two * b - taylor_two * c - d) / taylor_two =
        (taylor_two * a - taylor_two * b - taylor_two * c - d) * taylor_two.inverse
    taylor3_sub_mul(taylor_two * a - taylor_two * b - taylor_two * c, d, taylor_two.inverse)
    ((taylor_two * a - taylor_two * b - taylor_two * c) - d) * taylor_two.inverse =
        (taylor_two * a - taylor_two * b - taylor_two * c) * taylor_two.inverse -
            d * taylor_two.inverse
    taylor_two * a - taylor_two * b - taylor_two * c - d =
        (taylor_two * a - taylor_two * b - taylor_two * c) - d
    (taylor_two * a - taylor_two * b - taylor_two * c - d) * taylor_two.inverse =
        (taylor_two * a - taylor_two * b - taylor_two * c) * taylor_two.inverse -
            d * taylor_two.inverse
    taylor3_sub_mul(taylor_two * a - taylor_two * b, taylor_two * c, taylor_two.inverse)
    ((taylor_two * a - taylor_two * b) - taylor_two * c) * taylor_two.inverse =
        (taylor_two * a - taylor_two * b) * taylor_two.inverse -
            (taylor_two * c) * taylor_two.inverse
    taylor_two * a - taylor_two * b - taylor_two * c =
        (taylor_two * a - taylor_two * b) - taylor_two * c
    (taylor_two * a - taylor_two * b - taylor_two * c) * taylor_two.inverse =
        (taylor_two * a - taylor_two * b) * taylor_two.inverse -
            (taylor_two * c) * taylor_two.inverse
    taylor3_sub_mul(taylor_two * a, taylor_two * b, taylor_two.inverse)
    (taylor_two * a - taylor_two * b) * taylor_two.inverse =
        (taylor_two * a) * taylor_two.inverse - (taylor_two * b) * taylor_two.inverse
    real_mul_inverse_comm(taylor_two, a)
    (taylor_two * a) * taylor_two.inverse = a
    real_mul_inverse_comm(taylor_two, b)
    (taylor_two * b) * taylor_two.inverse = b
    real_mul_inverse_comm(taylor_two, c)
    (taylor_two * c) * taylor_two.inverse = c
    (taylor_two * a - taylor_two * b - taylor_two * c - d) * taylor_two.inverse =
        a - b - c - d * taylor_two.inverse
    d * taylor_two.inverse = d / taylor_two
    a - b - c - d * taylor_two.inverse = a - b - c - d / taylor_two
    (taylor_two * a - taylor_two * b - taylor_two * c - d) * taylor_two.inverse =
        a - b - c - d / taylor_two
    (taylor_two * a - taylor_two * b - taylor_two * c - d) / taylor_two =
        a - b - c - d / taylor_two
}

/// A product divided by three and then by two equals the product divided by six.
theorem taylor3_div_six_prod(x: Real, y: Real) {
    ((x / taylor_three) * y) / taylor_two = (x / taylor_six) * y
} by {
    taylor_three_not_zero
    taylor_two_not_zero
    ((x / taylor_three) * y) / taylor_two =
        ((x / taylor_three) * y) * taylor_two.inverse
    x / taylor_three = x * taylor_three.inverse
    ((x / taylor_three) * y) = (x * taylor_three.inverse) * y
    ((x / taylor_three) * y) * taylor_two.inverse =
        ((x * taylor_three.inverse) * y) * taylor_two.inverse
    mul_assoc(x, taylor_three.inverse, y)
    (x * taylor_three.inverse) * y = x * (taylor_three.inverse * y)
    ((x * taylor_three.inverse) * y) * taylor_two.inverse =
        (x * (taylor_three.inverse * y)) * taylor_two.inverse
    mul_assoc(x, taylor_three.inverse * y, taylor_two.inverse)
    (x * (taylor_three.inverse * y)) * taylor_two.inverse =
        x * ((taylor_three.inverse * y) * taylor_two.inverse)
    mul_assoc(taylor_three.inverse, y, taylor_two.inverse)
    (taylor_three.inverse * y) * taylor_two.inverse =
        taylor_three.inverse * (y * taylor_two.inverse)
    real_mul_comm(y, taylor_two.inverse)
    y * taylor_two.inverse = taylor_two.inverse * y
    taylor_three.inverse * (y * taylor_two.inverse) =
        taylor_three.inverse * (taylor_two.inverse * y)
    mul_assoc(taylor_three.inverse, taylor_two.inverse, y)
    taylor_three.inverse * (taylor_two.inverse * y) =
        (taylor_three.inverse * taylor_two.inverse) * y
    (taylor_three.inverse * y) * taylor_two.inverse =
        (taylor_three.inverse * taylor_two.inverse) * y
    x * ((taylor_three.inverse * y) * taylor_two.inverse) =
        x * ((taylor_three.inverse * taylor_two.inverse) * y)
    mul_assoc(x, taylor_three.inverse * taylor_two.inverse, y)
    x * ((taylor_three.inverse * taylor_two.inverse) * y) =
        (x * (taylor_three.inverse * taylor_two.inverse)) * y
    inverse_dist(taylor_two, taylor_three)
    (taylor_two * taylor_three).inverse = taylor_three.inverse * taylor_two.inverse
    x * (taylor_three.inverse * taylor_two.inverse) = x * (taylor_two * taylor_three).inverse
    taylor_six = taylor_two * taylor_three
    (taylor_two * taylor_three).inverse = taylor_six.inverse
    x * (taylor_three.inverse * taylor_two.inverse) = x * taylor_six.inverse
    (x * (taylor_three.inverse * taylor_two.inverse)) * y = (x * taylor_six.inverse) * y
    ((x / taylor_three) * y) * taylor_two.inverse = (x * taylor_six.inverse) * y
    x * taylor_six.inverse = x / taylor_six
    (x * taylor_six.inverse) * y = (x / taylor_six) * y
    ((x / taylor_three) * y) * taylor_two.inverse = (x / taylor_six) * y
    ((x / taylor_three) * y) / taylor_two = (x / taylor_six) * y
}

/// If the derivative of the auxiliary function of the third-order Taylor
/// theorem vanishes at c, the third-order Taylor formula holds at b.
theorem taylor3_conclusion_from_derivative(f: Real -> Real, df: Real -> Real, ddf: Real -> Real, dddf: Real -> Real, a: Real, b: Real, c: Real) {
    a < c and c < b and taylor3_aux_derivative(f, df, ddf, dddf, a, b, c) = Real.0
    implies f(b) = f(a) + df(a) * (b - a) + ddf(a) * (b - a).pow(Nat.2) / taylor_two +
        (dddf(c) / taylor_six) * (b - a).pow(Nat.3)
} by {
    if a < c and c < b and taylor3_aux_derivative(f, df, ddf, dddf, a, b, c) = Real.0 {
        taylor3_aux_derivative(f, df, ddf, dddf, a, b, c) =
            (b - c).pow(Nat.2) * (taylor_three * taylor3_remainder_coeff(f, df, ddf, a, b) - dddf(c))
        (b - c).pow(Nat.2) * (taylor_three * taylor3_remainder_coeff(f, df, ddf, a, b) - dddf(c)) = Real.0
        lt_imp_ne_symm(c, b)
        b != c
        sub_ne_zero_of_ne(b, c)
        b - c != Real.0
        mul_not_zero[Real](b - c, b - c)
        (b - c) * (b - c) != Real.0
        real_pow_two_eq_mul(b - c)
        (b - c).pow(Nat.2) = (b - c) * (b - c)
        (b - c).pow(Nat.2) != Real.0
        mul_zero_right(taylor_three * taylor3_remainder_coeff(f, df, ddf, a, b) - dddf(c))
        (taylor_three * taylor3_remainder_coeff(f, df, ddf, a, b) - dddf(c)) * Real.0 = Real.0
        mul_left_cancel((b - c).pow(Nat.2),
            taylor_three * taylor3_remainder_coeff(f, df, ddf, a, b) - dddf(c), Real.0)
        taylor_three * taylor3_remainder_coeff(f, df, ddf, a, b) - dddf(c) = Real.0
        sub_zero_imp_eq(taylor_three * taylor3_remainder_coeff(f, df, ddf, a, b), dddf(c))
        taylor_three * taylor3_remainder_coeff(f, df, ddf, a, b) = dddf(c)
        taylor_three_not_zero
        taylor_three != Real.0
        mul_inverse(taylor_three)
        taylor_three * taylor_three.inverse = Real.1
        (taylor_three * taylor3_remainder_coeff(f, df, ddf, a, b)) * taylor_three.inverse =
            dddf(c) * taylor_three.inverse
        mul_assoc(taylor_three, taylor3_remainder_coeff(f, df, ddf, a, b), taylor_three.inverse)
        (taylor_three * taylor3_remainder_coeff(f, df, ddf, a, b)) * taylor_three.inverse =
            taylor_three * (taylor3_remainder_coeff(f, df, ddf, a, b) * taylor_three.inverse)
        real_mul_comm(taylor3_remainder_coeff(f, df, ddf, a, b), taylor_three.inverse)
        taylor3_remainder_coeff(f, df, ddf, a, b) * taylor_three.inverse =
            taylor_three.inverse * taylor3_remainder_coeff(f, df, ddf, a, b)
        mul_assoc(taylor_three, taylor_three.inverse, taylor3_remainder_coeff(f, df, ddf, a, b))
        taylor_three * (taylor_three.inverse * taylor3_remainder_coeff(f, df, ddf, a, b)) =
            (taylor_three * taylor_three.inverse) * taylor3_remainder_coeff(f, df, ddf, a, b)
        (taylor_three * taylor_three.inverse) * taylor3_remainder_coeff(f, df, ddf, a, b) =
            Real.1 * taylor3_remainder_coeff(f, df, ddf, a, b)
        mul_one_left(taylor3_remainder_coeff(f, df, ddf, a, b))
        Real.1 * taylor3_remainder_coeff(f, df, ddf, a, b) = taylor3_remainder_coeff(f, df, ddf, a, b)
        taylor_three * (taylor_three.inverse * taylor3_remainder_coeff(f, df, ddf, a, b)) =
            taylor3_remainder_coeff(f, df, ddf, a, b)
        (taylor_three * taylor3_remainder_coeff(f, df, ddf, a, b)) * taylor_three.inverse =
            taylor3_remainder_coeff(f, df, ddf, a, b)
        taylor3_remainder_coeff(f, df, ddf, a, b) = dddf(c) * taylor_three.inverse
        taylor3_remainder_coeff(f, df, ddf, a, b) = dddf(c) / taylor_three
        lt_trans[Real](a, c, b)
        a < b
        lt_imp_ne_symm(a, b)
        b != a
        sub_ne_zero_of_ne(b, a)
        b - a != Real.0
        field_square_nonzero[Real](b - a)
        (b - a) * (b - a) != Real.0
        mul_not_zero[Real](b - a, (b - a) * (b - a))
        (b - a) * ((b - a) * (b - a)) != Real.0
        real_pow_three_eq_mul(b - a)
        (b - a).pow(Nat.3) = (b - a) * (b - a) * (b - a)
        (b - a).pow(Nat.3) != Real.0
        taylor3_remainder_coeff(f, df, ddf, a, b) =
            taylor3_doubled_delta(f, df, ddf, a, b) / (b - a).pow(Nat.3)
        div_mul_cancel_denominator(taylor3_doubled_delta(f, df, ddf, a, b), (b - a).pow(Nat.3))
        (taylor3_doubled_delta(f, df, ddf, a, b) / (b - a).pow(Nat.3)) * (b - a).pow(Nat.3) =
            taylor3_doubled_delta(f, df, ddf, a, b)
        taylor3_remainder_coeff(f, df, ddf, a, b) * (b - a).pow(Nat.3) =
            taylor3_doubled_delta(f, df, ddf, a, b)
        taylor3_doubled_delta(f, df, ddf, a, b) =
            taylor_two * f(b) - taylor_two * f(a) - taylor_two * df(a) * (b - a) -
                ddf(a) * (b - a).pow(Nat.2)
        taylor_two * f(b) - taylor_two * f(a) - taylor_two * df(a) * (b - a) -
            ddf(a) * (b - a).pow(Nat.2) = (dddf(c) / taylor_three) * (b - a).pow(Nat.3)
        taylor_two * f(b) - taylor_two * f(a) - taylor_two * df(a) * (b - a) -
            ddf(a) * (b - a).pow(Nat.2) = (dddf(c) / taylor_three) * (b - a).pow(Nat.3)
        taylor_two_not_zero
        taylor_two != Real.0
        (taylor_two * f(b) - taylor_two * f(a) - taylor_two * df(a) * (b - a) -
            ddf(a) * (b - a).pow(Nat.2)) / taylor_two =
            ((dddf(c) / taylor_three) * (b - a).pow(Nat.3)) / taylor_two
        taylor3_div_two_chain(f(b), f(a), df(a) * (b - a), ddf(a) * (b - a).pow(Nat.2))
        (taylor_two * f(b) - taylor_two * f(a) - taylor_two * (df(a) * (b - a)) -
            ddf(a) * (b - a).pow(Nat.2)) / taylor_two =
            f(b) - f(a) - df(a) * (b - a) - ddf(a) * (b - a).pow(Nat.2) / taylor_two
        mul_assoc(taylor_two, df(a), b - a)
        taylor_two * df(a) * (b - a) = taylor_two * (df(a) * (b - a))
        (taylor_two * f(b) - taylor_two * f(a) - taylor_two * df(a) * (b - a) -
            ddf(a) * (b - a).pow(Nat.2)) / taylor_two =
            (taylor_two * f(b) - taylor_two * f(a) - taylor_two * (df(a) * (b - a)) -
                ddf(a) * (b - a).pow(Nat.2)) / taylor_two
        (taylor_two * f(b) - taylor_two * f(a) - taylor_two * df(a) * (b - a) -
            ddf(a) * (b - a).pow(Nat.2)) / taylor_two =
            f(b) - f(a) - df(a) * (b - a) - ddf(a) * (b - a).pow(Nat.2) / taylor_two
        taylor3_div_six_prod(dddf(c), (b - a).pow(Nat.3))
        ((dddf(c) / taylor_three) * (b - a).pow(Nat.3)) / taylor_two =
            (dddf(c) / taylor_six) * (b - a).pow(Nat.3)
        f(b) - f(a) - df(a) * (b - a) - ddf(a) * (b - a).pow(Nat.2) / taylor_two =
            (dddf(c) / taylor_six) * (b - a).pow(Nat.3)
        taylor3_sub_add_cancel3(f(b), f(a), df(a) * (b - a),
            ddf(a) * (b - a).pow(Nat.2) / taylor_two)
        (f(b) - f(a) - df(a) * (b - a) - ddf(a) * (b - a).pow(Nat.2) / taylor_two) +
            f(a) + df(a) * (b - a) + ddf(a) * (b - a).pow(Nat.2) / taylor_two = f(b)
        (f(b) - f(a) - df(a) * (b - a) - ddf(a) * (b - a).pow(Nat.2) / taylor_two) +
            (f(a) + df(a) * (b - a) + ddf(a) * (b - a).pow(Nat.2) / taylor_two) =
            (dddf(c) / taylor_six) * (b - a).pow(Nat.3) +
                (f(a) + df(a) * (b - a) + ddf(a) * (b - a).pow(Nat.2) / taylor_two)
        f(b) = (dddf(c) / taylor_six) * (b - a).pow(Nat.3) +
            (f(a) + df(a) * (b - a) + ddf(a) * (b - a).pow(Nat.2) / taylor_two)
        f(b) = f(a) + df(a) * (b - a) + ddf(a) * (b - a).pow(Nat.2) / taylor_two +
            (dddf(c) / taylor_six) * (b - a).pow(Nat.3)
    }
}

/// Taylor's theorem of order two with Lagrange remainder, in the
/// iterated-derivative notation: if f has three derivatives everywhere, then
/// the value at b equals the second-order Taylor polynomial at a plus the
/// remainder f'''(c) (b - a)^3 / 6 for some interior c.
theorem taylor_general_two(f: Real -> Real, df: Real -> Real, ddf: Real -> Real, dddf: Real -> Real, a: Real, b: Real) {
    a < b and is_derivative_fn(f, df) and is_derivative_fn(df, ddf) and is_derivative_fn(ddf, dddf)
    implies exists(c: Real) {
        a < c and c < b and
        f(b) = taylor_poly(f, taylor_chain3(f, df, ddf, dddf), a, b, Nat.2) +
            taylor_remainder(f, taylor_chain3(f, df, ddf, dddf), a, b, c, Nat.2)
    }
} by {
    if a < b and is_derivative_fn(f, df) and is_derivative_fn(df, ddf) and is_derivative_fn(ddf, dddf) {
        taylor3_aux_continuous_on_closed(f, df, ddf, dddf, a, b, a, b)
        continuous_on_closed(taylor3_aux(f, df, ddf, a, b), a, b)
        taylor3_aux_is_derivative_fn(f, df, ddf, dddf, a, b)
        is_derivative_fn(taylor3_aux(f, df, ddf, a, b),
            taylor3_aux_derivative(f, df, ddf, dddf, a, b))
        is_derivative_fn_imp_is_derivative_on_open(taylor3_aux(f, df, ddf, a, b),
            taylor3_aux_derivative(f, df, ddf, dddf, a, b), a, b)
        is_derivative_on_open(taylor3_aux(f, df, ddf, a, b),
            taylor3_aux_derivative(f, df, ddf, dddf, a, b), a, b)
        is_derivative_on_open_imp_differentiable_on_open(taylor3_aux(f, df, ddf, a, b),
            taylor3_aux_derivative(f, df, ddf, dddf, a, b), a, b)
        differentiable_on_open(taylor3_aux(f, df, ddf, a, b), a, b)
        taylor3_aux_endpoints_equal(f, df, ddf, a, b)
        taylor3_aux(f, df, ddf, a, b, a) = taylor3_aux(f, df, ddf, a, b, b)
        rolle_theorem(taylor3_aux(f, df, ddf, a, b), a, b)
        let c1: Real satisfy {
            a < c1 and c1 < b and
            has_derivative_at(taylor3_aux(f, df, ddf, a, b), c1, Real.0)
        }
        a < c1 and c1 < b
        has_derivative_at(taylor3_aux(f, df, ddf, a, b), c1, Real.0)
        is_derivative_fn_at(taylor3_aux(f, df, ddf, a, b),
            taylor3_aux_derivative(f, df, ddf, dddf, a, b), c1)
        has_derivative_at(taylor3_aux(f, df, ddf, a, b), c1,
            taylor3_aux_derivative(f, df, ddf, dddf, a, b, c1))
        has_derivative_at_unique(taylor3_aux(f, df, ddf, a, b), c1, Real.0,
            taylor3_aux_derivative(f, df, ddf, dddf, a, b, c1))
        Real.0 = taylor3_aux_derivative(f, df, ddf, dddf, a, b, c1)
        taylor3_aux_derivative(f, df, ddf, dddf, a, b, c1) = Real.0
        taylor3_conclusion_from_derivative(f, df, ddf, dddf, a, b, c1)
        f(b) = f(a) + df(a) * (b - a) + ddf(a) * (b - a).pow(Nat.2) / taylor_two +
            (dddf(c1) / taylor_six) * (b - a).pow(Nat.3)
        taylor_poly(f, taylor_chain3(f, df, ddf, dddf), a, b, Nat.2) =
            partial(taylor_term(f, taylor_chain3(f, df, ddf, dddf), a, b), Nat.3)
        partial_three_sum(taylor_term(f, taylor_chain3(f, df, ddf, dddf), a, b))
        partial(taylor_term(f, taylor_chain3(f, df, ddf, dddf), a, b), Nat.3) =
            taylor_term(f, taylor_chain3(f, df, ddf, dddf), a, b, Nat.0) +
            taylor_term(f, taylor_chain3(f, df, ddf, dddf), a, b, Nat.1) +
            taylor_term(f, taylor_chain3(f, df, ddf, dddf), a, b, Nat.2)
        taylor_poly(f, taylor_chain3(f, df, ddf, dddf), a, b, Nat.2) =
            taylor_term(f, taylor_chain3(f, df, ddf, dddf), a, b, Nat.0) +
            taylor_term(f, taylor_chain3(f, df, ddf, dddf), a, b, Nat.1) +
            taylor_term(f, taylor_chain3(f, df, ddf, dddf), a, b, Nat.2)
        taylor_term(f, taylor_chain3(f, df, ddf, dddf), a, b, Nat.0) =
            (iterated_derivative(f, taylor_chain3(f, df, ddf, dddf), Nat.0, a) /
                from_nat[Real](Nat.0.factorial)) * (b - a).pow(Nat.0)
        iterated_derivative(f, taylor_chain3(f, df, ddf, dddf), Nat.0, a) =
            taylor_chain3(f, df, ddf, dddf, Nat.0, a)
        taylor_chain3_zero(f, df, ddf, dddf)
        taylor_chain3(f, df, ddf, dddf, Nat.0) = f
        taylor_chain3(f, df, ddf, dddf, Nat.0, a) = f(a)
        iterated_derivative(f, taylor_chain3(f, df, ddf, dddf), Nat.0, a) = f(a)
        from_nat_factorial_zero_real
        from_nat[Real](Nat.0.factorial) = Real.1
        real_pow_zero_eq(b - a)
        (b - a).pow(Nat.0) = Real.1
        taylor_term(f, taylor_chain3(f, df, ddf, dddf), a, b, Nat.0) = (f(a) / Real.1) * Real.1
        real_div_one(f(a))
        f(a) / Real.1 = f(a)
        (f(a) / Real.1) * Real.1 = f(a) * Real.1
        mul_one_right(f(a))
        f(a) * Real.1 = f(a)
        (f(a) / Real.1) * Real.1 = f(a)
        taylor_term(f, taylor_chain3(f, df, ddf, dddf), a, b, Nat.0) = f(a)
        taylor_term(f, taylor_chain3(f, df, ddf, dddf), a, b, Nat.1) =
            (iterated_derivative(f, taylor_chain3(f, df, ddf, dddf), Nat.1, a) /
                from_nat[Real](Nat.1.factorial)) * (b - a).pow(Nat.1)
        iterated_derivative(f, taylor_chain3(f, df, ddf, dddf), Nat.1, a) =
            taylor_chain3(f, df, ddf, dddf, Nat.1, a)
        taylor_chain3_one(f, df, ddf, dddf)
        taylor_chain3(f, df, ddf, dddf, Nat.1) = df
        taylor_chain3(f, df, ddf, dddf, Nat.1, a) = df(a)
        iterated_derivative(f, taylor_chain3(f, df, ddf, dddf), Nat.1, a) = df(a)
        from_nat_factorial_one_real
        from_nat[Real](Nat.1.factorial) = Real.1
        real_pow_one_eq(b - a)
        (b - a).pow(Nat.1) = b - a
        taylor_term(f, taylor_chain3(f, df, ddf, dddf), a, b, Nat.1) =
            (df(a) / Real.1) * (b - a)
        real_div_one(df(a))
        df(a) / Real.1 = df(a)
        (df(a) / Real.1) * (b - a) = df(a) * (b - a)
        taylor_term(f, taylor_chain3(f, df, ddf, dddf), a, b, Nat.1) = df(a) * (b - a)
        taylor_term(f, taylor_chain3(f, df, ddf, dddf), a, b, Nat.2) =
            (iterated_derivative(f, taylor_chain3(f, df, ddf, dddf), Nat.2, a) /
                from_nat[Real](Nat.2.factorial)) * (b - a).pow(Nat.2)
        iterated_derivative(f, taylor_chain3(f, df, ddf, dddf), Nat.2, a) =
            taylor_chain3(f, df, ddf, dddf, Nat.2, a)
        taylor_chain3_two(f, df, ddf, dddf)
        taylor_chain3(f, df, ddf, dddf, Nat.2) = ddf
        taylor_chain3(f, df, ddf, dddf, Nat.2, a) = ddf(a)
        iterated_derivative(f, taylor_chain3(f, df, ddf, dddf), Nat.2, a) = ddf(a)
        from_nat_factorial_two_real
        from_nat[Real](Nat.2.factorial) = taylor_two
        taylor_term(f, taylor_chain3(f, df, ddf, dddf), a, b, Nat.2) =
            (ddf(a) / taylor_two) * (b - a).pow(Nat.2)
        taylor_poly(f, taylor_chain3(f, df, ddf, dddf), a, b, Nat.2) =
            f(a) + df(a) * (b - a) + (ddf(a) / taylor_two) * (b - a).pow(Nat.2)
        taylor_remainder(f, taylor_chain3(f, df, ddf, dddf), a, b, c1, Nat.2) =
            (iterated_derivative(f, taylor_chain3(f, df, ddf, dddf), Nat.2.suc, c1) /
                from_nat[Real](Nat.2.suc.factorial)) * (b - a).pow(Nat.2.suc)
        Nat.2.suc = Nat.3
        taylor_remainder(f, taylor_chain3(f, df, ddf, dddf), a, b, c1, Nat.2) =
            (iterated_derivative(f, taylor_chain3(f, df, ddf, dddf), Nat.3, c1) /
                from_nat[Real](Nat.3.factorial)) * (b - a).pow(Nat.3)
        iterated_derivative(f, taylor_chain3(f, df, ddf, dddf), Nat.3, c1) =
            taylor_chain3(f, df, ddf, dddf, Nat.3, c1)
        taylor_chain3_three(f, df, ddf, dddf)
        taylor_chain3(f, df, ddf, dddf, Nat.3) = dddf
        taylor_chain3(f, df, ddf, dddf, Nat.3, c1) = dddf(c1)
        iterated_derivative(f, taylor_chain3(f, df, ddf, dddf), Nat.3, c1) = dddf(c1)
        from_nat_factorial_three_real
        from_nat[Real](Nat.3.factorial) = taylor_six
        taylor_remainder(f, taylor_chain3(f, df, ddf, dddf), a, b, c1, Nat.2) =
            (dddf(c1) / taylor_six) * (b - a).pow(Nat.3)
        ddf(a) / taylor_two = ddf(a) * taylor_two.inverse
        (ddf(a) / taylor_two) * (b - a).pow(Nat.2) =
            (ddf(a) * taylor_two.inverse) * (b - a).pow(Nat.2)
        mul_assoc(ddf(a), taylor_two.inverse, (b - a).pow(Nat.2))
        (ddf(a) * taylor_two.inverse) * (b - a).pow(Nat.2) =
            ddf(a) * (taylor_two.inverse * (b - a).pow(Nat.2))
        real_mul_comm(taylor_two.inverse, (b - a).pow(Nat.2))
        taylor_two.inverse * (b - a).pow(Nat.2) =
            (b - a).pow(Nat.2) * taylor_two.inverse
        ddf(a) * (taylor_two.inverse * (b - a).pow(Nat.2)) =
            ddf(a) * ((b - a).pow(Nat.2) * taylor_two.inverse)
        mul_assoc(ddf(a), (b - a).pow(Nat.2), taylor_two.inverse)
        ddf(a) * ((b - a).pow(Nat.2) * taylor_two.inverse) =
            (ddf(a) * (b - a).pow(Nat.2)) * taylor_two.inverse
        (ddf(a) / taylor_two) * (b - a).pow(Nat.2) =
            (ddf(a) * (b - a).pow(Nat.2)) * taylor_two.inverse
        ddf(a) * (b - a).pow(Nat.2) / taylor_two =
            (ddf(a) * (b - a).pow(Nat.2)) * taylor_two.inverse
        (ddf(a) / taylor_two) * (b - a).pow(Nat.2) =
            ddf(a) * (b - a).pow(Nat.2) / taylor_two
        f(a) + df(a) * (b - a) + (ddf(a) / taylor_two) * (b - a).pow(Nat.2) +
            (dddf(c1) / taylor_six) * (b - a).pow(Nat.3) =
            f(a) + df(a) * (b - a) + ddf(a) * (b - a).pow(Nat.2) / taylor_two +
                (dddf(c1) / taylor_six) * (b - a).pow(Nat.3)
        taylor_poly(f, taylor_chain3(f, df, ddf, dddf), a, b, Nat.2) +
            taylor_remainder(f, taylor_chain3(f, df, ddf, dddf), a, b, c1, Nat.2) =
            f(a) + df(a) * (b - a) + ddf(a) * (b - a).pow(Nat.2) / taylor_two +
                (dddf(c1) / taylor_six) * (b - a).pow(Nat.3)
        f(b) = taylor_poly(f, taylor_chain3(f, df, ddf, dddf), a, b, Nat.2) +
            taylor_remainder(f, taylor_chain3(f, df, ddf, dddf), a, b, c1, Nat.2)
        exists(c2: Real) {
            a < c2 and c2 < b and
            f(b) = taylor_poly(f, taylor_chain3(f, df, ddf, dddf), a, b, Nat.2) +
                taylor_remainder(f, taylor_chain3(f, df, ddf, dddf), a, b, c2, Nat.2)
        }
    }
}

/// The real number four, written as 2 * 2.
let taylor_four: Real = taylor_two * taylor_two

/// The real number twenty-four, written as 4 * 6.
let taylor_24: Real = taylor_four * taylor_six

/// The real number four is nonzero.
theorem taylor_four_not_zero {
    taylor_four != Real.0
} by {
    taylor_four = taylor_two * taylor_two
    taylor_two_not_zero
    mul_not_zero[Real](taylor_two, taylor_two)
    taylor_two * taylor_two != Real.0
    taylor_four != Real.0
}

/// The real number twenty-four is nonzero.
theorem taylor_24_not_zero {
    taylor_24 != Real.0
} by {
    taylor_24 = taylor_four * taylor_six
    taylor_four_not_zero
    taylor_six_not_zero
    mul_not_zero[Real](taylor_four, taylor_six)
    taylor_four * taylor_six != Real.0
    taylor_24 != Real.0
}

/// The image of four in the reals is four.
theorem from_nat_four_real {
    from_nat[Real](Nat.4) = taylor_four
} by {
    from_nat_mul[Real](Nat.2, Nat.2)
    from_nat[Real](Nat.2 * Nat.2) = from_nat[Real](Nat.2) * from_nat[Real](Nat.2)
    from_nat_two_real
    from_nat[Real](Nat.2) = taylor_two
    from_nat[Real](Nat.2 * Nat.2) = taylor_two * taylor_two
    Nat.2 * Nat.2 = Nat.4
    from_nat[Real](Nat.4) = taylor_two * taylor_two
    taylor_four = taylor_two * taylor_two
    from_nat[Real](Nat.4) = taylor_four
}

/// The image of twenty-four in the reals is twenty-four.
theorem from_nat_24_real {
    from_nat[Real](Nat.4 * Nat.6) = taylor_24
} by {
    from_nat_mul[Real](Nat.4, Nat.6)
    from_nat[Real](Nat.4 * Nat.6) = from_nat[Real](Nat.4) * from_nat[Real](Nat.6)
    from_nat_four_real
    from_nat[Real](Nat.4) = taylor_four
    from_nat_six_real
    from_nat[Real](Nat.6) = taylor_six
    from_nat[Real](Nat.4 * Nat.6) = taylor_four * taylor_six
    taylor_24 = taylor_four * taylor_six
    from_nat[Real](Nat.4 * Nat.6) = taylor_24
}

/// The factorial of four is four times six.
theorem factorial_four_nat {
    Nat.4.factorial = Nat.4 * Nat.6
} by {
    factorial_step(Nat.3)
    Nat.3.suc.factorial = Nat.3.suc * Nat.3.factorial
    Nat.3.suc = Nat.4
    Nat.4.factorial = Nat.4 * Nat.3.factorial
    factorial_three_nat
    Nat.3.factorial = Nat.6
    Nat.4.factorial = Nat.4 * Nat.6
}

/// The image of the factorial of four in the reals is twenty-four.
theorem from_nat_factorial_four_real {
    from_nat[Real](Nat.4.factorial) = taylor_24
} by {
    factorial_four_nat
    Nat.4.factorial = Nat.4 * Nat.6
    from_nat[Real](Nat.4.factorial) = from_nat[Real](Nat.4 * Nat.6)
    from_nat_24_real
    from_nat[Real](Nat.4 * Nat.6) = taylor_24
    from_nat[Real](Nat.4.factorial) = taylor_24
}

/// The sixfold fourth-order increment of f over [a, b]: six times the value at
/// b minus the third-order Taylor polynomial of f at a.
define taylor4_doubled_delta(f: Real -> Real, df: Real -> Real, ddf: Real -> Real, dddf: Real -> Real, a: Real, b: Real) -> Real {
    taylor_six * f(b) - taylor_six * f(a) - taylor_six * df(a) * (b - a) -
        taylor_three * ddf(a) * (b - a).pow(Nat.2) - dddf(a) * (b - a).pow(Nat.3)
}

/// The coefficient of the quartic remainder term: the sixfold fourth-order
/// increment divided by (b - a)^4.
define taylor4_remainder_coeff(f: Real -> Real, df: Real -> Real, ddf: Real -> Real, dddf: Real -> Real, a: Real, b: Real) -> Real {
    taylor4_doubled_delta(f, df, ddf, dddf, a, b) / (b - a).pow(Nat.4)
}

/// The auxiliary function for the fourth-order Taylor theorem, scaled by six:
/// it vanishes at a and at b, so Rolle's theorem applies to it.
define taylor4_aux(f: Real -> Real, df: Real -> Real, ddf: Real -> Real, dddf: Real -> Real, a: Real, b: Real, x: Real) -> Real {
    taylor_six * f(b) - taylor_six * f(x) - taylor_six * df(x) * (b - x) -
        taylor_three * ddf(x) * (b - x).pow(Nat.2) - dddf(x) * (b - x).pow(Nat.3) -
        taylor4_remainder_coeff(f, df, ddf, dddf, a, b) * (b - x).pow(Nat.4)
}

/// The derivative of the auxiliary function of the fourth-order Taylor theorem.
define taylor4_aux_derivative(f: Real -> Real, df: Real -> Real, ddf: Real -> Real, dddf: Real -> Real, dddd: Real -> Real, a: Real, b: Real, x: Real) -> Real {
    (b - x).pow(Nat.3) * (taylor_four * taylor4_remainder_coeff(f, df, ddf, dddf, a, b) - dddd(x))
}

/// The fourth power of the shift x ↦ (b - x)^4 has derivative -4 (b - x)^3 at
/// every point.
theorem taylor4_quartic_has_derivative_at(b: Real, x0: Real) {
    has_derivative_at(pointwise_mul(compose(square_real, affine_real(-Real.1, b)),
        compose(square_real, affine_real(-Real.1, b))), x0,
        -taylor_four * (b - x0).pow(Nat.3))
} by {
    taylor3_sq_has_derivative_at(b, x0)
    has_derivative_at(compose(square_real, affine_real(-Real.1, b)), x0,
        -taylor_two * (b - x0))
    derivative_pointwise_square(compose(square_real, affine_real(-Real.1, b)), x0,
        -taylor_two * (b - x0))
    has_derivative_at(pointwise_mul(compose(square_real, affine_real(-Real.1, b)),
        compose(square_real, affine_real(-Real.1, b))), x0,
        compose(square_real, affine_real(-Real.1, b), x0) * (-taylor_two * (b - x0)) +
            compose(square_real, affine_real(-Real.1, b), x0) * (-taylor_two * (b - x0)))
    compose(square_real, affine_real(-Real.1, b), x0) =
        square_real(affine_real(-Real.1, b, x0))
    square_real(affine_real(-Real.1, b, x0)) = square_real(b - x0)
    real_pow_two_eq_mul(b - x0)
    (b - x0).pow(Nat.2) = (b - x0) * (b - x0)
    square_real(b - x0) = (b - x0).pow(Nat.2)
    compose(square_real, affine_real(-Real.1, b), x0) = (b - x0).pow(Nat.2)
    mul_one_right((b - x0).pow(Nat.2) * (-taylor_two * (b - x0)))
    ((b - x0).pow(Nat.2) * (-taylor_two * (b - x0))) * Real.1 =
        (b - x0).pow(Nat.2) * (-taylor_two * (b - x0))
    mul_distrib_right((b - x0).pow(Nat.2) * (-taylor_two * (b - x0)), Real.1, Real.1)
    ((b - x0).pow(Nat.2) * (-taylor_two * (b - x0))) * (Real.1 + Real.1) =
        ((b - x0).pow(Nat.2) * (-taylor_two * (b - x0))) * Real.1 +
            ((b - x0).pow(Nat.2) * (-taylor_two * (b - x0))) * Real.1
    taylor_two = Real.1 + Real.1
    ((b - x0).pow(Nat.2) * (-taylor_two * (b - x0))) * taylor_two =
        ((b - x0).pow(Nat.2) * (-taylor_two * (b - x0))) * (Real.1 + Real.1)
    ((b - x0).pow(Nat.2) * (-taylor_two * (b - x0))) * taylor_two =
        (b - x0).pow(Nat.2) * (-taylor_two * (b - x0)) +
            (b - x0).pow(Nat.2) * (-taylor_two * (b - x0))
    real_mul_comm((b - x0).pow(Nat.2) * (-taylor_two * (b - x0)), taylor_two)
    ((b - x0).pow(Nat.2) * (-taylor_two * (b - x0))) * taylor_two =
        taylor_two * ((b - x0).pow(Nat.2) * (-taylor_two * (b - x0)))
    (b - x0).pow(Nat.2) * (-taylor_two * (b - x0)) +
        (b - x0).pow(Nat.2) * (-taylor_two * (b - x0)) =
        taylor_two * ((b - x0).pow(Nat.2) * (-taylor_two * (b - x0)))
    mul_assoc((b - x0).pow(Nat.2), -taylor_two, b - x0)
    (b - x0).pow(Nat.2) * (-taylor_two * (b - x0)) =
        ((b - x0).pow(Nat.2) * -taylor_two) * (b - x0)
    real_mul_comm((b - x0).pow(Nat.2), -taylor_two)
    (b - x0).pow(Nat.2) * -taylor_two = -taylor_two * (b - x0).pow(Nat.2)
    ((b - x0).pow(Nat.2) * -taylor_two) * (b - x0) =
        (-taylor_two * (b - x0).pow(Nat.2)) * (b - x0)
    (b - x0).pow(Nat.2) * (-taylor_two * (b - x0)) =
        (-taylor_two * (b - x0).pow(Nat.2)) * (b - x0)
    taylor_two * ((b - x0).pow(Nat.2) * (-taylor_two * (b - x0))) =
        taylor_two * ((-taylor_two * (b - x0).pow(Nat.2)) * (b - x0))
    mul_assoc(taylor_two, -taylor_two, (b - x0).pow(Nat.2))
    taylor_two * (-taylor_two * (b - x0).pow(Nat.2)) =
        (taylor_two * -taylor_two) * (b - x0).pow(Nat.2)
    mul_neg_right(taylor_two, taylor_two)
    taylor_two * -taylor_two = -(taylor_two * taylor_two)
    (taylor_two * -taylor_two) * (b - x0).pow(Nat.2) =
        -(taylor_two * taylor_two) * (b - x0).pow(Nat.2)
    taylor_four = taylor_two * taylor_two
    -(taylor_two * taylor_two) * (b - x0).pow(Nat.2) =
        -taylor_four * (b - x0).pow(Nat.2)
    (taylor_two * -taylor_two) * (b - x0).pow(Nat.2) =
        -taylor_four * (b - x0).pow(Nat.2)
    taylor_two * (-taylor_two * (b - x0).pow(Nat.2)) =
        -taylor_four * (b - x0).pow(Nat.2)
    taylor_two * ((-taylor_two * (b - x0).pow(Nat.2)) * (b - x0)) =
        (-taylor_four * (b - x0).pow(Nat.2)) * (b - x0)
    mul_assoc(-taylor_four, (b - x0).pow(Nat.2), b - x0)
    (-taylor_four * (b - x0).pow(Nat.2)) * (b - x0) =
        -taylor_four * ((b - x0).pow(Nat.2) * (b - x0))
    real_pow_three_eq_mul(b - x0)
    (b - x0).pow(Nat.3) = (b - x0) * (b - x0) * (b - x0)
    mul_assoc(b - x0, b - x0, b - x0)
    (b - x0) * (b - x0) * (b - x0) = (b - x0) * ((b - x0) * (b - x0))
    real_pow_two_eq_mul(b - x0)
    (b - x0).pow(Nat.2) = (b - x0) * (b - x0)
    (b - x0).pow(Nat.2) * (b - x0) = (b - x0).pow(Nat.3)
    -taylor_four * ((b - x0).pow(Nat.2) * (b - x0)) =
        -taylor_four * (b - x0).pow(Nat.3)
    taylor_two * ((-taylor_two * (b - x0).pow(Nat.2)) * (b - x0)) =
        -taylor_four * (b - x0).pow(Nat.3)
    compose(square_real, affine_real(-Real.1, b), x0) * (-taylor_two * (b - x0)) +
        compose(square_real, affine_real(-Real.1, b), x0) * (-taylor_two * (b - x0)) =
        -taylor_four * (b - x0).pow(Nat.3)
    has_derivative_at(pointwise_mul(compose(square_real, affine_real(-Real.1, b)),
        compose(square_real, affine_real(-Real.1, b))), x0,
        -taylor_four * (b - x0).pow(Nat.3))
}

/// The left half of the pointwise fourth-order auxiliary function: the constant,
/// minus six times f and minus six df times (b - x).
define taylor4_aux_left(f: Real -> Real, df: Real -> Real, a: Real, b: Real) -> Real -> Real {
    pointwise_add(
        pointwise_add(constant[Real, Real](taylor_six * f(b)),
            pointwise_neg(pointwise_mul(constant[Real, Real](taylor_six), f))),
        pointwise_neg(pointwise_mul(pointwise_mul(constant[Real, Real](taylor_six), df),
            affine_real(-Real.1, b))))
}

/// The right half of the pointwise fourth-order auxiliary function: minus three
/// ddf times (b - x)^2, minus dddf times (b - x)^3 and minus the remainder
/// coefficient times (b - x)^4.
define taylor4_aux_right(f: Real -> Real, df: Real -> Real, ddf: Real -> Real, dddf: Real -> Real, a: Real, b: Real) -> Real -> Real {
    pointwise_add(
        pointwise_add(
            pointwise_neg(pointwise_mul(pointwise_mul(constant[Real, Real](taylor_three), ddf),
                compose(square_real, affine_real(-Real.1, b)))),
            pointwise_neg(pointwise_mul(dddf, compose(cube_real, affine_real(-Real.1, b))))),
        pointwise_neg(pointwise_mul(constant[Real, Real](taylor4_remainder_coeff(f, df, ddf, dddf, a, b)),
            pointwise_mul(compose(square_real, affine_real(-Real.1, b)),
                compose(square_real, affine_real(-Real.1, b))))))
}

/// The auxiliary function of the fourth-order Taylor theorem as a pointwise
/// combination of f, df, ddf, dddf, constants, an affine function, the square,
/// the cube and the fourth power.
define taylor4_aux_pointwise(f: Real -> Real, df: Real -> Real, ddf: Real -> Real, dddf: Real -> Real, a: Real, b: Real) -> Real -> Real {
    pointwise_add(taylor4_aux_left(f, df, a, b), taylor4_aux_right(f, df, ddf, dddf, a, b))
}
/// The auxiliary function of the fourth-order Taylor theorem agrees with its
/// pointwise combination.
theorem taylor4_aux_eq_pointwise(f: Real -> Real, df: Real -> Real, ddf: Real -> Real, dddf: Real -> Real, a: Real, b: Real) {
    taylor4_aux(f, df, ddf, dddf, a, b) = taylor4_aux_pointwise(f, df, ddf, dddf, a, b)
} by {
    forall(x: Real) {
        taylor4_aux(f, df, ddf, dddf, a, b, x) = taylor_six * f(b) - taylor_six * f(x) -
            taylor_six * df(x) * (b - x) - taylor_three * ddf(x) * (b - x).pow(Nat.2) -
            dddf(x) * (b - x).pow(Nat.3) -
            taylor4_remainder_coeff(f, df, ddf, dddf, a, b) * (b - x).pow(Nat.4)
        constant[Real, Real](taylor_six * f(b), x) = taylor_six * f(b)
        pointwise_mul(constant[Real, Real](taylor_six), f, x) =
            constant[Real, Real](taylor_six, x) * f(x)
        constant[Real, Real](taylor_six, x) = taylor_six
        pointwise_mul(constant[Real, Real](taylor_six), f, x) = taylor_six * f(x)
        pointwise_neg(pointwise_mul(constant[Real, Real](taylor_six), f), x) =
            -(pointwise_mul(constant[Real, Real](taylor_six), f, x))
        pointwise_neg(pointwise_mul(constant[Real, Real](taylor_six), f), x) =
            -(taylor_six * f(x))
        pointwise_mul(constant[Real, Real](taylor_six), df, x) =
            constant[Real, Real](taylor_six, x) * df(x)
        pointwise_mul(constant[Real, Real](taylor_six), df, x) = taylor_six * df(x)
        pointwise_mul(constant[Real, Real](taylor_three), ddf, x) =
            constant[Real, Real](taylor_three, x) * ddf(x)
        constant[Real, Real](taylor_three, x) = taylor_three
        pointwise_mul(constant[Real, Real](taylor_three), ddf, x) = taylor_three * ddf(x)
        affine_real(-Real.1, b, x) = -Real.1 * x + b
        mul_neg_left(Real.1, x)
        -Real.1 * x = -(Real.1 * x)
        mul_one_left(x)
        Real.1 * x = x
        -(Real.1 * x) = -x
        -Real.1 * x = -x
        -x + b = b - x
        affine_real(-Real.1, b, x) = b - x
        pointwise_mul(pointwise_mul(constant[Real, Real](taylor_six), df),
            affine_real(-Real.1, b), x) =
            pointwise_mul(constant[Real, Real](taylor_six), df, x) *
                affine_real(-Real.1, b, x)
        pointwise_mul(pointwise_mul(constant[Real, Real](taylor_six), df),
            affine_real(-Real.1, b), x) = taylor_six * df(x) * (b - x)
        pointwise_neg(pointwise_mul(pointwise_mul(constant[Real, Real](taylor_six), df),
            affine_real(-Real.1, b)), x) =
            -(pointwise_mul(pointwise_mul(constant[Real, Real](taylor_six), df),
                affine_real(-Real.1, b), x))
        pointwise_neg(pointwise_mul(pointwise_mul(constant[Real, Real](taylor_six), df),
            affine_real(-Real.1, b)), x) = -(taylor_six * df(x) * (b - x))
        taylor4_aux_left(f, df, a, b) = pointwise_add(
            pointwise_add(constant[Real, Real](taylor_six * f(b)),
                pointwise_neg(pointwise_mul(constant[Real, Real](taylor_six), f))),
            pointwise_neg(pointwise_mul(pointwise_mul(constant[Real, Real](taylor_six), df),
                affine_real(-Real.1, b))))
        taylor4_aux_left(f, df, a, b, x) = pointwise_add(
            pointwise_add(constant[Real, Real](taylor_six * f(b)),
                pointwise_neg(pointwise_mul(constant[Real, Real](taylor_six), f))),
            pointwise_neg(pointwise_mul(pointwise_mul(constant[Real, Real](taylor_six), df),
                affine_real(-Real.1, b))), x)
        taylor4_aux_left(f, df, a, b, x) =
            pointwise_add(constant[Real, Real](taylor_six * f(b)),
                pointwise_neg(pointwise_mul(constant[Real, Real](taylor_six), f)), x) +
            pointwise_neg(pointwise_mul(pointwise_mul(constant[Real, Real](taylor_six), df),
                affine_real(-Real.1, b)), x)
        pointwise_add(constant[Real, Real](taylor_six * f(b)),
            pointwise_neg(pointwise_mul(constant[Real, Real](taylor_six), f)), x) =
            constant[Real, Real](taylor_six * f(b), x) +
                pointwise_neg(pointwise_mul(constant[Real, Real](taylor_six), f), x)
        pointwise_add(constant[Real, Real](taylor_six * f(b)),
            pointwise_neg(pointwise_mul(constant[Real, Real](taylor_six), f)), x) =
            taylor_six * f(b) + -(taylor_six * f(x))
        taylor4_aux_left(f, df, a, b, x) =
            taylor_six * f(b) + -(taylor_six * f(x)) + -(taylor_six * df(x) * (b - x))
        taylor4_aux_left(f, df, a, b, x) =
            taylor_six * f(b) - taylor_six * f(x) - taylor_six * df(x) * (b - x)
        compose(square_real, affine_real(-Real.1, b), x) =
            square_real(affine_real(-Real.1, b, x))
        square_real(affine_real(-Real.1, b, x)) = square_real(b - x)
        real_pow_two_eq_mul(b - x)
        (b - x).pow(Nat.2) = (b - x) * (b - x)
        square_real(b - x) = (b - x).pow(Nat.2)
        compose(square_real, affine_real(-Real.1, b), x) = (b - x).pow(Nat.2)
        pointwise_mul(pointwise_mul(constant[Real, Real](taylor_three), ddf),
            compose(square_real, affine_real(-Real.1, b)), x) =
            pointwise_mul(constant[Real, Real](taylor_three), ddf, x) *
                compose(square_real, affine_real(-Real.1, b), x)
        pointwise_mul(pointwise_mul(constant[Real, Real](taylor_three), ddf),
            compose(square_real, affine_real(-Real.1, b)), x) =
            taylor_three * ddf(x) * (b - x).pow(Nat.2)
        pointwise_neg(pointwise_mul(pointwise_mul(constant[Real, Real](taylor_three), ddf),
            compose(square_real, affine_real(-Real.1, b))), x) =
            -(pointwise_mul(pointwise_mul(constant[Real, Real](taylor_three), ddf),
                compose(square_real, affine_real(-Real.1, b)), x))
        pointwise_neg(pointwise_mul(pointwise_mul(constant[Real, Real](taylor_three), ddf),
            compose(square_real, affine_real(-Real.1, b))), x) =
            -(taylor_three * ddf(x) * (b - x).pow(Nat.2))
        compose(cube_real, affine_real(-Real.1, b), x) =
            cube_real(affine_real(-Real.1, b, x))
        cube_real(affine_real(-Real.1, b, x)) = cube_real(b - x)
        real_pow_three_eq_mul(b - x)
        (b - x).pow(Nat.3) = (b - x) * (b - x) * (b - x)
        cube_real(b - x) = (b - x).pow(Nat.3)
        compose(cube_real, affine_real(-Real.1, b), x) = (b - x).pow(Nat.3)
        pointwise_mul(dddf, compose(cube_real, affine_real(-Real.1, b)), x) =
            dddf(x) * compose(cube_real, affine_real(-Real.1, b), x)
        pointwise_mul(dddf, compose(cube_real, affine_real(-Real.1, b)), x) =
            dddf(x) * (b - x).pow(Nat.3)
        pointwise_neg(pointwise_mul(dddf, compose(cube_real, affine_real(-Real.1, b))), x) =
            -(pointwise_mul(dddf, compose(cube_real, affine_real(-Real.1, b)), x))
        pointwise_neg(pointwise_mul(dddf, compose(cube_real, affine_real(-Real.1, b))), x) =
            -(dddf(x) * (b - x).pow(Nat.3))
        pointwise_mul(compose(square_real, affine_real(-Real.1, b)),
            compose(square_real, affine_real(-Real.1, b)), x) =
            compose(square_real, affine_real(-Real.1, b), x) *
                compose(square_real, affine_real(-Real.1, b), x)
        pow_add(b - x, Nat.2, Nat.2)
        (b - x).pow(Nat.2 + Nat.2) = (b - x).pow(Nat.2) * (b - x).pow(Nat.2)
        Nat.2 + Nat.2 = Nat.4
        (b - x).pow(Nat.4) = (b - x).pow(Nat.2) * (b - x).pow(Nat.2)
        pointwise_mul(compose(square_real, affine_real(-Real.1, b)),
            compose(square_real, affine_real(-Real.1, b)), x) = (b - x).pow(Nat.4)
        pointwise_mul(constant[Real, Real](taylor4_remainder_coeff(f, df, ddf, dddf, a, b)),
            pointwise_mul(compose(square_real, affine_real(-Real.1, b)),
                compose(square_real, affine_real(-Real.1, b))), x) =
            constant[Real, Real](taylor4_remainder_coeff(f, df, ddf, dddf, a, b), x) *
                pointwise_mul(compose(square_real, affine_real(-Real.1, b)),
                    compose(square_real, affine_real(-Real.1, b)), x)
        constant[Real, Real](taylor4_remainder_coeff(f, df, ddf, dddf, a, b), x) =
            taylor4_remainder_coeff(f, df, ddf, dddf, a, b)
        pointwise_mul(constant[Real, Real](taylor4_remainder_coeff(f, df, ddf, dddf, a, b)),
            pointwise_mul(compose(square_real, affine_real(-Real.1, b)),
                compose(square_real, affine_real(-Real.1, b))), x) =
            taylor4_remainder_coeff(f, df, ddf, dddf, a, b) * (b - x).pow(Nat.4)
        pointwise_neg(pointwise_mul(constant[Real, Real](taylor4_remainder_coeff(f, df, ddf, dddf, a, b)),
            pointwise_mul(compose(square_real, affine_real(-Real.1, b)),
                compose(square_real, affine_real(-Real.1, b)))), x) =
            -(pointwise_mul(constant[Real, Real](taylor4_remainder_coeff(f, df, ddf, dddf, a, b)),
                pointwise_mul(compose(square_real, affine_real(-Real.1, b)),
                    compose(square_real, affine_real(-Real.1, b))), x))
        pointwise_neg(pointwise_mul(constant[Real, Real](taylor4_remainder_coeff(f, df, ddf, dddf, a, b)),
            pointwise_mul(compose(square_real, affine_real(-Real.1, b)),
                compose(square_real, affine_real(-Real.1, b)))), x) =
            -(taylor4_remainder_coeff(f, df, ddf, dddf, a, b) * (b - x).pow(Nat.4))
        taylor4_aux_right(f, df, ddf, dddf, a, b) = pointwise_add(
            pointwise_add(
                pointwise_neg(pointwise_mul(pointwise_mul(constant[Real, Real](taylor_three), ddf),
                    compose(square_real, affine_real(-Real.1, b)))),
                pointwise_neg(pointwise_mul(dddf, compose(cube_real, affine_real(-Real.1, b))))),
            pointwise_neg(pointwise_mul(constant[Real, Real](taylor4_remainder_coeff(f, df, ddf, dddf, a, b)),
                pointwise_mul(compose(square_real, affine_real(-Real.1, b)),
                    compose(square_real, affine_real(-Real.1, b))))))
        taylor4_aux_right(f, df, ddf, dddf, a, b, x) = pointwise_add(
            pointwise_add(
                pointwise_neg(pointwise_mul(pointwise_mul(constant[Real, Real](taylor_three), ddf),
                    compose(square_real, affine_real(-Real.1, b)))),
                pointwise_neg(pointwise_mul(dddf, compose(cube_real, affine_real(-Real.1, b))))),
            pointwise_neg(pointwise_mul(constant[Real, Real](taylor4_remainder_coeff(f, df, ddf, dddf, a, b)),
                pointwise_mul(compose(square_real, affine_real(-Real.1, b)),
                    compose(square_real, affine_real(-Real.1, b))))), x)
        taylor4_aux_right(f, df, ddf, dddf, a, b, x) =
            pointwise_add(
                pointwise_neg(pointwise_mul(pointwise_mul(constant[Real, Real](taylor_three), ddf),
                    compose(square_real, affine_real(-Real.1, b)))),
                pointwise_neg(pointwise_mul(dddf, compose(cube_real, affine_real(-Real.1, b)))), x) +
            pointwise_neg(pointwise_mul(constant[Real, Real](taylor4_remainder_coeff(f, df, ddf, dddf, a, b)),
                pointwise_mul(compose(square_real, affine_real(-Real.1, b)),
                    compose(square_real, affine_real(-Real.1, b)))), x)
        pointwise_add(
            pointwise_neg(pointwise_mul(pointwise_mul(constant[Real, Real](taylor_three), ddf),
                compose(square_real, affine_real(-Real.1, b)))),
            pointwise_neg(pointwise_mul(dddf, compose(cube_real, affine_real(-Real.1, b)))), x) =
            pointwise_neg(pointwise_mul(pointwise_mul(constant[Real, Real](taylor_three), ddf),
                compose(square_real, affine_real(-Real.1, b))), x) +
                pointwise_neg(pointwise_mul(dddf, compose(cube_real, affine_real(-Real.1, b))), x)
        taylor4_aux_right(f, df, ddf, dddf, a, b, x) =
            (-(taylor_three * ddf(x) * (b - x).pow(Nat.2)) + -(dddf(x) * (b - x).pow(Nat.3))) +
                -(taylor4_remainder_coeff(f, df, ddf, dddf, a, b) * (b - x).pow(Nat.4))
        taylor4_aux_pointwise(f, df, ddf, dddf, a, b) =
            pointwise_add(taylor4_aux_left(f, df, a, b), taylor4_aux_right(f, df, ddf, dddf, a, b))
        taylor4_aux_pointwise(f, df, ddf, dddf, a, b, x) =
            pointwise_add(taylor4_aux_left(f, df, a, b), taylor4_aux_right(f, df, ddf, dddf, a, b), x)
        pointwise_add(taylor4_aux_left(f, df, a, b), taylor4_aux_right(f, df, ddf, dddf, a, b), x) =
            taylor4_aux_left(f, df, a, b, x) + taylor4_aux_right(f, df, ddf, dddf, a, b, x)
        taylor4_aux_pointwise(f, df, ddf, dddf, a, b, x) =
            taylor4_aux_left(f, df, a, b, x) + taylor4_aux_right(f, df, ddf, dddf, a, b, x)
        taylor4_aux_pointwise(f, df, ddf, dddf, a, b, x) =
            (taylor_six * f(b) - taylor_six * f(x) - taylor_six * df(x) * (b - x)) +
                ((-(taylor_three * ddf(x) * (b - x).pow(Nat.2)) + -(dddf(x) * (b - x).pow(Nat.3))) +
                    -(taylor4_remainder_coeff(f, df, ddf, dddf, a, b) * (b - x).pow(Nat.4)))
        (taylor_six * f(b) - taylor_six * f(x) - taylor_six * df(x) * (b - x)) +
            ((-(taylor_three * ddf(x) * (b - x).pow(Nat.2)) + -(dddf(x) * (b - x).pow(Nat.3))) +
                -(taylor4_remainder_coeff(f, df, ddf, dddf, a, b) * (b - x).pow(Nat.4))) =
            ((taylor_six * f(b) - taylor_six * f(x) - taylor_six * df(x) * (b - x)) +
                (-(taylor_three * ddf(x) * (b - x).pow(Nat.2)) + -(dddf(x) * (b - x).pow(Nat.3)))) +
                -(taylor4_remainder_coeff(f, df, ddf, dddf, a, b) * (b - x).pow(Nat.4))
        ((taylor_six * f(b) - taylor_six * f(x) - taylor_six * df(x) * (b - x)) +
            (-(taylor_three * ddf(x) * (b - x).pow(Nat.2)) + -(dddf(x) * (b - x).pow(Nat.3)))) +
            -(taylor4_remainder_coeff(f, df, ddf, dddf, a, b) * (b - x).pow(Nat.4)) =
            ((taylor_six * f(b) - taylor_six * f(x) - taylor_six * df(x) * (b - x)) +
                -(taylor_three * ddf(x) * (b - x).pow(Nat.2))) + -(dddf(x) * (b - x).pow(Nat.3)) +
                -(taylor4_remainder_coeff(f, df, ddf, dddf, a, b) * (b - x).pow(Nat.4))
        (taylor_six * f(b) - taylor_six * f(x) - taylor_six * df(x) * (b - x)) +
            -(taylor_three * ddf(x) * (b - x).pow(Nat.2)) = taylor_six * f(b) - taylor_six * f(x) -
                taylor_six * df(x) * (b - x) - taylor_three * ddf(x) * (b - x).pow(Nat.2)
        (taylor_six * f(b) - taylor_six * f(x) - taylor_six * df(x) * (b - x) -
            taylor_three * ddf(x) * (b - x).pow(Nat.2)) + -(dddf(x) * (b - x).pow(Nat.3)) =
            taylor_six * f(b) - taylor_six * f(x) - taylor_six * df(x) * (b - x) -
                taylor_three * ddf(x) * (b - x).pow(Nat.2) - dddf(x) * (b - x).pow(Nat.3)
        (taylor_six * f(b) - taylor_six * f(x) - taylor_six * df(x) * (b - x) -
            taylor_three * ddf(x) * (b - x).pow(Nat.2) - dddf(x) * (b - x).pow(Nat.3)) +
            -(taylor4_remainder_coeff(f, df, ddf, dddf, a, b) * (b - x).pow(Nat.4)) =
            taylor_six * f(b) - taylor_six * f(x) - taylor_six * df(x) * (b - x) -
                taylor_three * ddf(x) * (b - x).pow(Nat.2) - dddf(x) * (b - x).pow(Nat.3) -
                taylor4_remainder_coeff(f, df, ddf, dddf, a, b) * (b - x).pow(Nat.4)
        taylor4_aux_pointwise(f, df, ddf, dddf, a, b, x) =
            taylor_six * f(b) - taylor_six * f(x) - taylor_six * df(x) * (b - x) -
                taylor_three * ddf(x) * (b - x).pow(Nat.2) - dddf(x) * (b - x).pow(Nat.3) -
                taylor4_remainder_coeff(f, df, ddf, dddf, a, b) * (b - x).pow(Nat.4)
        taylor4_aux(f, df, ddf, dddf, a, b, x) = taylor4_aux_pointwise(f, df, ddf, dddf, a, b, x)
    }
    function_extensionality(taylor4_aux(f, df, ddf, dddf, a, b), taylor4_aux_pointwise(f, df, ddf, dddf, a, b))
    taylor4_aux(f, df, ddf, dddf, a, b) = taylor4_aux_pointwise(f, df, ddf, dddf, a, b)
}
/// The auxiliary function of the fourth-order Taylor theorem is continuous on
/// every closed interval when f, df, ddf and dddf are differentiable everywhere.
theorem taylor4_aux_continuous_on_closed(f: Real -> Real, df: Real -> Real, ddf: Real -> Real, dddf: Real -> Real, dddd: Real -> Real, a: Real, b: Real, u: Real, v: Real) {
    is_derivative_fn(f, df) and is_derivative_fn(df, ddf) and is_derivative_fn(ddf, dddf) and is_derivative_fn(dddf, dddd)
    implies continuous_on_closed(taylor4_aux(f, df, ddf, dddf, a, b), u, v)
} by {
    if is_derivative_fn(f, df) and is_derivative_fn(df, ddf) and is_derivative_fn(ddf, dddf) and is_derivative_fn(dddf, dddd) {
        forall(x: Real) {
            if closed_interval_set(u, v).contains(x) {
                is_derivative_fn_imp_continuous_at(f, df, x)
                continuous_at(f, x)
                is_derivative_fn_imp_continuous_at(df, ddf, x)
                continuous_at(df, x)
                is_derivative_fn_imp_continuous_at(ddf, dddf, x)
                continuous_at(ddf, x)
                is_derivative_fn_imp_continuous_at(dddf, dddd, x)
                continuous_at(dddf, x)
                constant_function_is_continuous_at(taylor_six * f(b), x)
                continuous_at(constant[Real, Real](taylor_six * f(b)), x)
                constant_function_is_continuous_at(taylor_six, x)
                continuous_at(constant[Real, Real](taylor_six), x)
                continuous_at_pointwise_mul(constant[Real, Real](taylor_six), f, x)
                continuous_at(pointwise_mul(constant[Real, Real](taylor_six), f), x)
                continuous_at_pointwise_neg(pointwise_mul(constant[Real, Real](taylor_six), f), x)
                continuous_at(pointwise_neg(pointwise_mul(constant[Real, Real](taylor_six), f)), x)
                continuous_at_pointwise_add(constant[Real, Real](taylor_six * f(b)),
                    pointwise_neg(pointwise_mul(constant[Real, Real](taylor_six), f)), x)
                continuous_at(pointwise_add(constant[Real, Real](taylor_six * f(b)),
                    pointwise_neg(pointwise_mul(constant[Real, Real](taylor_six), f))), x)
                continuous_at_affine_real(-Real.1, b, x)
                continuous_at(affine_real(-Real.1, b), x)
                continuous_at_pointwise_mul(pointwise_mul(constant[Real, Real](taylor_six), df),
                    affine_real(-Real.1, b), x)
                continuous_at(pointwise_mul(pointwise_mul(constant[Real, Real](taylor_six), df),
                    affine_real(-Real.1, b)), x)
                continuous_at_pointwise_neg(pointwise_mul(pointwise_mul(constant[Real, Real](taylor_six), df),
                    affine_real(-Real.1, b)), x)
                continuous_at(pointwise_neg(pointwise_mul(pointwise_mul(constant[Real, Real](taylor_six), df),
                    affine_real(-Real.1, b))), x)
                continuous_at_pointwise_add(pointwise_add(constant[Real, Real](taylor_six * f(b)),
                        pointwise_neg(pointwise_mul(constant[Real, Real](taylor_six), f))),
                    pointwise_neg(pointwise_mul(pointwise_mul(constant[Real, Real](taylor_six), df),
                        affine_real(-Real.1, b))), x)
                continuous_at(pointwise_add(pointwise_add(constant[Real, Real](taylor_six * f(b)),
                        pointwise_neg(pointwise_mul(constant[Real, Real](taylor_six), f))),
                    pointwise_neg(pointwise_mul(pointwise_mul(constant[Real, Real](taylor_six), df),
                        affine_real(-Real.1, b)))), x)
                taylor4_aux_left(f, df, a, b) = pointwise_add(
                    pointwise_add(constant[Real, Real](taylor_six * f(b)),
                        pointwise_neg(pointwise_mul(constant[Real, Real](taylor_six), f))),
                    pointwise_neg(pointwise_mul(pointwise_mul(constant[Real, Real](taylor_six), df),
                        affine_real(-Real.1, b))))
                function_eq_transport_predicate_rev(
                    function(h: Real -> Real) { continuous_at(h, x) },
                    taylor4_aux_left(f, df, a, b),
                    pointwise_add(pointwise_add(constant[Real, Real](taylor_six * f(b)),
                            pointwise_neg(pointwise_mul(constant[Real, Real](taylor_six), f))),
                        pointwise_neg(pointwise_mul(pointwise_mul(constant[Real, Real](taylor_six), df),
                            affine_real(-Real.1, b)))))
                continuous_at(taylor4_aux_left(f, df, a, b), x)
                constant_function_is_continuous_at(taylor_three, x)
                continuous_at(constant[Real, Real](taylor_three), x)
                continuous_at_pointwise_mul(constant[Real, Real](taylor_three), ddf, x)
                continuous_at(pointwise_mul(constant[Real, Real](taylor_three), ddf), x)
                continuous_at_square_real(affine_real(-Real.1, b, x))
                continuous_at(square_real, affine_real(-Real.1, b, x))
                continuous_at_compose(square_real, affine_real(-Real.1, b), x)
                continuous_at(compose(square_real, affine_real(-Real.1, b)), x)
                continuous_at_pointwise_mul(pointwise_mul(constant[Real, Real](taylor_three), ddf),
                    compose(square_real, affine_real(-Real.1, b)), x)
                continuous_at(pointwise_mul(pointwise_mul(constant[Real, Real](taylor_three), ddf),
                    compose(square_real, affine_real(-Real.1, b))), x)
                continuous_at_pointwise_neg(pointwise_mul(pointwise_mul(constant[Real, Real](taylor_three), ddf),
                    compose(square_real, affine_real(-Real.1, b))), x)
                continuous_at(pointwise_neg(pointwise_mul(pointwise_mul(constant[Real, Real](taylor_three), ddf),
                    compose(square_real, affine_real(-Real.1, b)))), x)
                continuous_at_cube_real(affine_real(-Real.1, b, x))
                continuous_at(cube_real, affine_real(-Real.1, b, x))
                continuous_at_compose(cube_real, affine_real(-Real.1, b), x)
                continuous_at(compose(cube_real, affine_real(-Real.1, b)), x)
                continuous_at_pointwise_mul(dddf, compose(cube_real, affine_real(-Real.1, b)), x)
                continuous_at(pointwise_mul(dddf, compose(cube_real, affine_real(-Real.1, b))), x)
                continuous_at_pointwise_neg(pointwise_mul(dddf, compose(cube_real, affine_real(-Real.1, b))), x)
                continuous_at(pointwise_neg(pointwise_mul(dddf, compose(cube_real, affine_real(-Real.1, b)))), x)
                continuous_at_pointwise_add(pointwise_neg(pointwise_mul(pointwise_mul(constant[Real, Real](taylor_three), ddf),
                        compose(square_real, affine_real(-Real.1, b)))),
                    pointwise_neg(pointwise_mul(dddf, compose(cube_real, affine_real(-Real.1, b)))), x)
                continuous_at(pointwise_add(pointwise_neg(pointwise_mul(pointwise_mul(constant[Real, Real](taylor_three), ddf),
                            compose(square_real, affine_real(-Real.1, b)))),
                        pointwise_neg(pointwise_mul(dddf, compose(cube_real, affine_real(-Real.1, b))))), x)
                continuous_at_pointwise_mul(compose(square_real, affine_real(-Real.1, b)),
                    compose(square_real, affine_real(-Real.1, b)), x)
                continuous_at(pointwise_mul(compose(square_real, affine_real(-Real.1, b)),
                    compose(square_real, affine_real(-Real.1, b))), x)
                constant_function_is_continuous_at(taylor4_remainder_coeff(f, df, ddf, dddf, a, b), x)
                continuous_at(constant[Real, Real](taylor4_remainder_coeff(f, df, ddf, dddf, a, b)), x)
                continuous_at_pointwise_mul(constant[Real, Real](taylor4_remainder_coeff(f, df, ddf, dddf, a, b)),
                    pointwise_mul(compose(square_real, affine_real(-Real.1, b)),
                        compose(square_real, affine_real(-Real.1, b))), x)
                continuous_at(pointwise_mul(constant[Real, Real](taylor4_remainder_coeff(f, df, ddf, dddf, a, b)),
                    pointwise_mul(compose(square_real, affine_real(-Real.1, b)),
                        compose(square_real, affine_real(-Real.1, b)))), x)
                continuous_at_pointwise_neg(pointwise_mul(constant[Real, Real](taylor4_remainder_coeff(f, df, ddf, dddf, a, b)),
                    pointwise_mul(compose(square_real, affine_real(-Real.1, b)),
                        compose(square_real, affine_real(-Real.1, b)))), x)
                continuous_at(pointwise_neg(pointwise_mul(constant[Real, Real](taylor4_remainder_coeff(f, df, ddf, dddf, a, b)),
                    pointwise_mul(compose(square_real, affine_real(-Real.1, b)),
                        compose(square_real, affine_real(-Real.1, b))))), x)
                continuous_at_pointwise_add(pointwise_add(pointwise_neg(pointwise_mul(pointwise_mul(constant[Real, Real](taylor_three), ddf),
                            compose(square_real, affine_real(-Real.1, b)))),
                        pointwise_neg(pointwise_mul(dddf, compose(cube_real, affine_real(-Real.1, b))))),
                    pointwise_neg(pointwise_mul(constant[Real, Real](taylor4_remainder_coeff(f, df, ddf, dddf, a, b)),
                        pointwise_mul(compose(square_real, affine_real(-Real.1, b)),
                            compose(square_real, affine_real(-Real.1, b))))), x)
                continuous_at(pointwise_add(pointwise_add(pointwise_neg(pointwise_mul(pointwise_mul(constant[Real, Real](taylor_three), ddf),
                                compose(square_real, affine_real(-Real.1, b)))),
                            pointwise_neg(pointwise_mul(dddf, compose(cube_real, affine_real(-Real.1, b))))),
                        pointwise_neg(pointwise_mul(constant[Real, Real](taylor4_remainder_coeff(f, df, ddf, dddf, a, b)),
                            pointwise_mul(compose(square_real, affine_real(-Real.1, b)),
                                compose(square_real, affine_real(-Real.1, b)))))), x)
                taylor4_aux_right(f, df, ddf, dddf, a, b) = pointwise_add(
                    pointwise_add(
                        pointwise_neg(pointwise_mul(pointwise_mul(constant[Real, Real](taylor_three), ddf),
                            compose(square_real, affine_real(-Real.1, b)))),
                        pointwise_neg(pointwise_mul(dddf, compose(cube_real, affine_real(-Real.1, b))))),
                    pointwise_neg(pointwise_mul(constant[Real, Real](taylor4_remainder_coeff(f, df, ddf, dddf, a, b)),
                        pointwise_mul(compose(square_real, affine_real(-Real.1, b)),
                            compose(square_real, affine_real(-Real.1, b))))))
                function_eq_transport_predicate_rev(
                    function(h: Real -> Real) { continuous_at(h, x) },
                    taylor4_aux_right(f, df, ddf, dddf, a, b),
                    pointwise_add(pointwise_add(
                            pointwise_neg(pointwise_mul(pointwise_mul(constant[Real, Real](taylor_three), ddf),
                                compose(square_real, affine_real(-Real.1, b)))),
                            pointwise_neg(pointwise_mul(dddf, compose(cube_real, affine_real(-Real.1, b))))),
                        pointwise_neg(pointwise_mul(constant[Real, Real](taylor4_remainder_coeff(f, df, ddf, dddf, a, b)),
                            pointwise_mul(compose(square_real, affine_real(-Real.1, b)),
                                compose(square_real, affine_real(-Real.1, b)))))))
                continuous_at(taylor4_aux_right(f, df, ddf, dddf, a, b), x)
                taylor4_aux_pointwise(f, df, ddf, dddf, a, b) =
                    pointwise_add(taylor4_aux_left(f, df, a, b), taylor4_aux_right(f, df, ddf, dddf, a, b))
                continuous_at_pointwise_add(taylor4_aux_left(f, df, a, b),
                    taylor4_aux_right(f, df, ddf, dddf, a, b), x)
                continuous_at(pointwise_add(taylor4_aux_left(f, df, a, b),
                    taylor4_aux_right(f, df, ddf, dddf, a, b)), x)
                function_eq_transport_predicate_rev(
                    function(h: Real -> Real) { continuous_at(h, x) },
                    taylor4_aux_pointwise(f, df, ddf, dddf, a, b),
                    pointwise_add(taylor4_aux_left(f, df, a, b), taylor4_aux_right(f, df, ddf, dddf, a, b)))
                continuous_at(taylor4_aux_pointwise(f, df, ddf, dddf, a, b), x)
                taylor4_aux_eq_pointwise(f, df, ddf, dddf, a, b)
                taylor4_aux(f, df, ddf, dddf, a, b) = taylor4_aux_pointwise(f, df, ddf, dddf, a, b)
                function_eq_transport_predicate_rev(
                    function(h: Real -> Real) { continuous_at(h, x) },
                    taylor4_aux(f, df, ddf, dddf, a, b),
                    taylor4_aux_pointwise(f, df, ddf, dddf, a, b))
                continuous_at(taylor4_aux(f, df, ddf, dddf, a, b), x)
            }
        }
        continuous_on_closed(taylor4_aux(f, df, ddf, dddf, a, b), u, v) = forall(x: Real) {
            closed_interval_set(u, v).contains(x) implies
                continuous_at(taylor4_aux(f, df, ddf, dddf, a, b), x)
        }
        continuous_on_closed(taylor4_aux(f, df, ddf, dddf, a, b), u, v)
    }
}

/// The auxiliary function of the fourth-order Taylor theorem vanishes at a.
theorem taylor4_aux_at_a_zero(f: Real -> Real, df: Real -> Real, ddf: Real -> Real, dddf: Real -> Real, a: Real, b: Real) {
    a < b implies taylor4_aux(f, df, ddf, dddf, a, b, a) = Real.0
} by {
    if a < b {
        taylor4_aux(f, df, ddf, dddf, a, b, a) = taylor_six * f(b) - taylor_six * f(a) -
            taylor_six * df(a) * (b - a) - taylor_three * ddf(a) * (b - a).pow(Nat.2) -
            dddf(a) * (b - a).pow(Nat.3) -
            taylor4_remainder_coeff(f, df, ddf, dddf, a, b) * (b - a).pow(Nat.4)
        lt_imp_ne_symm(a, b)
        b != a
        sub_ne_zero_of_ne(b, a)
        b - a != Real.0
        field_square_nonzero[Real](b - a)
        (b - a) * (b - a) != Real.0
        mul_not_zero[Real]((b - a) * (b - a), (b - a) * (b - a))
        ((b - a) * (b - a)) * ((b - a) * (b - a)) != Real.0
        real_pow_four_eq_mul(b - a)
        (b - a).pow(Nat.4) = ((b - a) * (b - a)) * ((b - a) * (b - a))
        (b - a).pow(Nat.4) != Real.0
        taylor4_remainder_coeff(f, df, ddf, dddf, a, b) =
            taylor4_doubled_delta(f, df, ddf, dddf, a, b) / (b - a).pow(Nat.4)
        div_mul_cancel_denominator(taylor4_doubled_delta(f, df, ddf, dddf, a, b), (b - a).pow(Nat.4))
        (taylor4_doubled_delta(f, df, ddf, dddf, a, b) / (b - a).pow(Nat.4)) * (b - a).pow(Nat.4) =
            taylor4_doubled_delta(f, df, ddf, dddf, a, b)
        taylor4_remainder_coeff(f, df, ddf, dddf, a, b) * (b - a).pow(Nat.4) =
            taylor4_doubled_delta(f, df, ddf, dddf, a, b)
        taylor4_doubled_delta(f, df, ddf, dddf, a, b) =
            taylor_six * f(b) - taylor_six * f(a) - taylor_six * df(a) * (b - a) -
                taylor_three * ddf(a) * (b - a).pow(Nat.2) - dddf(a) * (b - a).pow(Nat.3)
        taylor4_aux(f, df, ddf, dddf, a, b, a) = taylor_six * f(b) - taylor_six * f(a) -
            taylor_six * df(a) * (b - a) - taylor_three * ddf(a) * (b - a).pow(Nat.2) -
            dddf(a) * (b - a).pow(Nat.3) -
            taylor4_doubled_delta(f, df, ddf, dddf, a, b)
        taylor_six * f(b) - taylor_six * f(a) - taylor_six * df(a) * (b - a) -
            taylor_three * ddf(a) * (b - a).pow(Nat.2) - dddf(a) * (b - a).pow(Nat.3) -
            (taylor_six * f(b) - taylor_six * f(a) - taylor_six * df(a) * (b - a) -
                taylor_three * ddf(a) * (b - a).pow(Nat.2) - dddf(a) * (b - a).pow(Nat.3)) = Real.0
        taylor4_aux(f, df, ddf, dddf, a, b, a) = Real.0
    }
}

/// The auxiliary function of the fourth-order Taylor theorem vanishes at b.
theorem taylor4_aux_at_b_zero(f: Real -> Real, df: Real -> Real, ddf: Real -> Real, dddf: Real -> Real, a: Real, b: Real) {
    taylor4_aux(f, df, ddf, dddf, a, b, b) = Real.0
} by {
    taylor4_aux(f, df, ddf, dddf, a, b, b) = taylor_six * f(b) - taylor_six * f(b) -
        taylor_six * df(b) * (b - b) - taylor_three * ddf(b) * (b - b).pow(Nat.2) -
        dddf(b) * (b - b).pow(Nat.3) -
        taylor4_remainder_coeff(f, df, ddf, dddf, a, b) * (b - b).pow(Nat.4)
    b - b = Real.0
    real_pow_two_eq_mul(b - b)
    (b - b).pow(Nat.2) = (b - b) * (b - b)
    (b - b) * (b - b) = Real.0 * Real.0
    mul_zero_left(Real.0)
    Real.0 * Real.0 = Real.0
    (b - b).pow(Nat.2) = Real.0
    real_pow_three_eq_mul(b - b)
    (b - b).pow(Nat.3) = Real.0
    real_pow_four_eq_mul(b - b)
    (b - b).pow(Nat.4) = Real.0
    taylor_six * df(b) * (b - b) = Real.0
    taylor_three * ddf(b) * (b - b).pow(Nat.2) = Real.0
    dddf(b) * (b - b).pow(Nat.3) = Real.0
    taylor4_remainder_coeff(f, df, ddf, dddf, a, b) * (b - b).pow(Nat.4) = Real.0
    taylor_six * f(b) - taylor_six * f(b) = Real.0
    taylor_six * f(b) - taylor_six * f(b) - Real.0 - Real.0 - Real.0 - Real.0 = Real.0
    taylor4_aux(f, df, ddf, dddf, a, b, b) = Real.0
}

/// The auxiliary function of the fourth-order Taylor theorem takes equal values
/// at the endpoints of the interval.
theorem taylor4_aux_endpoints_equal(f: Real -> Real, df: Real -> Real, ddf: Real -> Real, dddf: Real -> Real, a: Real, b: Real) {
    a < b implies taylor4_aux(f, df, ddf, dddf, a, b, a) = taylor4_aux(f, df, ddf, dddf, a, b, b)
} by {
    if a < b {
        taylor4_aux_at_a_zero(f, df, ddf, dddf, a, b)
        taylor4_aux(f, df, ddf, dddf, a, b, a) = Real.0
        taylor4_aux_at_b_zero(f, df, ddf, dddf, a, b)
        taylor4_aux(f, df, ddf, dddf, a, b, b) = Real.0
        taylor4_aux(f, df, ddf, dddf, a, b, a) = taylor4_aux(f, df, ddf, dddf, a, b, b)
    }
}

/// The auxiliary function of the fourth-order Taylor theorem has the expected
/// global derivative function.
theorem taylor4_aux_is_derivative_fn(f: Real -> Real, df: Real -> Real, ddf: Real -> Real, dddf: Real -> Real, dddd: Real -> Real, a: Real, b: Real) {
    is_derivative_fn(f, df) and is_derivative_fn(df, ddf) and is_derivative_fn(ddf, dddf) and is_derivative_fn(dddf, dddd)
    implies is_derivative_fn(taylor4_aux(f, df, ddf, dddf, a, b),
        taylor4_aux_derivative(f, df, ddf, dddf, dddd, a, b))
} by {
    if is_derivative_fn(f, df) and is_derivative_fn(df, ddf) and is_derivative_fn(ddf, dddf) and is_derivative_fn(dddf, dddd) {
        forall(x: Real) {
            is_derivative_fn_at(f, df, x)
            has_derivative_at(f, x, df(x))
            derivative_pointwise_const_mul(taylor_six, f, x, df(x))
            has_derivative_at(pointwise_mul(constant[Real, Real](taylor_six), f), x,
                taylor_six * df(x))
            derivative_pointwise_neg(pointwise_mul(constant[Real, Real](taylor_six), f),
                x, taylor_six * df(x))
            has_derivative_at(pointwise_neg(pointwise_mul(constant[Real, Real](taylor_six), f)),
                x, -(taylor_six * df(x)))
            constant_has_derivative_at(taylor_six * f(b), x)
            has_derivative_at(constant[Real, Real](taylor_six * f(b)), x, Real.0)
            derivative_pointwise_add(constant[Real, Real](taylor_six * f(b)),
                pointwise_neg(pointwise_mul(constant[Real, Real](taylor_six), f)),
                x, Real.0, -(taylor_six * df(x)))
            has_derivative_at(pointwise_add(constant[Real, Real](taylor_six * f(b)),
                pointwise_neg(pointwise_mul(constant[Real, Real](taylor_six), f))),
                x, Real.0 + -(taylor_six * df(x)))
            taylor4_aux_left(f, df, a, b) = pointwise_add(
                pointwise_add(constant[Real, Real](taylor_six * f(b)),
                    pointwise_neg(pointwise_mul(constant[Real, Real](taylor_six), f))),
                pointwise_neg(pointwise_mul(pointwise_mul(constant[Real, Real](taylor_six), df),
                    affine_real(-Real.1, b))))
            affine_real_has_derivative_at(-Real.1, b, x)
            has_derivative_at(affine_real(-Real.1, b), x, -Real.1)
            is_derivative_fn_at(df, ddf, x)
            has_derivative_at(df, x, ddf(x))
            derivative_pointwise_const_mul(taylor_six, df, x, ddf(x))
            has_derivative_at(pointwise_mul(constant[Real, Real](taylor_six), df), x,
                taylor_six * ddf(x))
            derivative_pointwise_mul(pointwise_mul(constant[Real, Real](taylor_six), df),
                affine_real(-Real.1, b), x, taylor_six * ddf(x), -Real.1)
            has_derivative_at(pointwise_mul(pointwise_mul(constant[Real, Real](taylor_six), df),
                affine_real(-Real.1, b)), x,
                pointwise_mul(constant[Real, Real](taylor_six), df, x) * -Real.1 +
                    affine_real(-Real.1, b, x) * (taylor_six * ddf(x)))
            derivative_pointwise_neg(pointwise_mul(pointwise_mul(constant[Real, Real](taylor_six), df),
                affine_real(-Real.1, b)), x,
                pointwise_mul(constant[Real, Real](taylor_six), df, x) * -Real.1 +
                    affine_real(-Real.1, b, x) * (taylor_six * ddf(x)))
            has_derivative_at(pointwise_neg(pointwise_mul(pointwise_mul(constant[Real, Real](taylor_six), df),
                affine_real(-Real.1, b))), x,
                -(pointwise_mul(constant[Real, Real](taylor_six), df, x) * -Real.1 +
                    affine_real(-Real.1, b, x) * (taylor_six * ddf(x))))
            derivative_pointwise_add(pointwise_add(constant[Real, Real](taylor_six * f(b)),
                    pointwise_neg(pointwise_mul(constant[Real, Real](taylor_six), f))),
                pointwise_neg(pointwise_mul(pointwise_mul(constant[Real, Real](taylor_six), df),
                    affine_real(-Real.1, b))),
                x, Real.0 + -(taylor_six * df(x)),
                -(pointwise_mul(constant[Real, Real](taylor_six), df, x) * -Real.1 +
                    affine_real(-Real.1, b, x) * (taylor_six * ddf(x))))
            has_derivative_at(taylor4_aux_left(f, df, a, b), x,
                (Real.0 + -(taylor_six * df(x))) +
                    -(pointwise_mul(constant[Real, Real](taylor_six), df, x) * -Real.1 +
                        affine_real(-Real.1, b, x) * (taylor_six * ddf(x))))
            is_derivative_fn_at(ddf, dddf, x)
            has_derivative_at(ddf, x, dddf(x))
            derivative_pointwise_const_mul(taylor_three, ddf, x, dddf(x))
            has_derivative_at(pointwise_mul(constant[Real, Real](taylor_three), ddf), x,
                taylor_three * dddf(x))
            taylor3_sq_has_derivative_at(b, x)
            has_derivative_at(compose(square_real, affine_real(-Real.1, b)), x,
                -taylor_two * (b - x))
            derivative_pointwise_mul(pointwise_mul(constant[Real, Real](taylor_three), ddf),
                compose(square_real, affine_real(-Real.1, b)), x,
                taylor_three * dddf(x), -taylor_two * (b - x))
            has_derivative_at(pointwise_mul(pointwise_mul(constant[Real, Real](taylor_three), ddf),
                compose(square_real, affine_real(-Real.1, b))), x,
                pointwise_mul(constant[Real, Real](taylor_three), ddf, x) *
                    (-taylor_two * (b - x)) +
                    compose(square_real, affine_real(-Real.1, b), x) *
                        (taylor_three * dddf(x)))
            derivative_pointwise_neg(pointwise_mul(pointwise_mul(constant[Real, Real](taylor_three), ddf),
                compose(square_real, affine_real(-Real.1, b))), x,
                pointwise_mul(constant[Real, Real](taylor_three), ddf, x) *
                    (-taylor_two * (b - x)) +
                    compose(square_real, affine_real(-Real.1, b), x) *
                        (taylor_three * dddf(x)))
            has_derivative_at(pointwise_neg(pointwise_mul(pointwise_mul(constant[Real, Real](taylor_three), ddf),
                compose(square_real, affine_real(-Real.1, b)))), x,
                -(pointwise_mul(constant[Real, Real](taylor_three), ddf, x) *
                    (-taylor_two * (b - x)) +
                    compose(square_real, affine_real(-Real.1, b), x) *
                        (taylor_three * dddf(x))))
            is_derivative_fn_at(dddf, dddd, x)
            has_derivative_at(dddf, x, dddd(x))
            taylor3_cube_has_derivative_at(b, x)
            has_derivative_at(compose(cube_real, affine_real(-Real.1, b)), x,
                -taylor_three * (b - x).pow(Nat.2))
            derivative_pointwise_mul(dddf, compose(cube_real, affine_real(-Real.1, b)),
                x, dddd(x), -taylor_three * (b - x).pow(Nat.2))
            has_derivative_at(pointwise_mul(dddf, compose(cube_real, affine_real(-Real.1, b))),
                x, dddf(x) * (-taylor_three * (b - x).pow(Nat.2)) +
                    compose(cube_real, affine_real(-Real.1, b), x) * dddd(x))
            derivative_pointwise_neg(pointwise_mul(dddf, compose(cube_real, affine_real(-Real.1, b))),
                x, dddf(x) * (-taylor_three * (b - x).pow(Nat.2)) +
                    compose(cube_real, affine_real(-Real.1, b), x) * dddd(x))
            has_derivative_at(pointwise_neg(pointwise_mul(dddf, compose(cube_real, affine_real(-Real.1, b)))),
                x, -(dddf(x) * (-taylor_three * (b - x).pow(Nat.2)) +
                    compose(cube_real, affine_real(-Real.1, b), x) * dddd(x)))
            taylor4_aux_right(f, df, ddf, dddf, a, b) = pointwise_add(
                pointwise_add(
                    pointwise_neg(pointwise_mul(pointwise_mul(constant[Real, Real](taylor_three), ddf),
                        compose(square_real, affine_real(-Real.1, b)))),
                    pointwise_neg(pointwise_mul(dddf, compose(cube_real, affine_real(-Real.1, b))))),
                pointwise_neg(pointwise_mul(constant[Real, Real](taylor4_remainder_coeff(f, df, ddf, dddf, a, b)),
                    pointwise_mul(compose(square_real, affine_real(-Real.1, b)),
                        compose(square_real, affine_real(-Real.1, b))))))
            derivative_pointwise_add(pointwise_neg(pointwise_mul(pointwise_mul(constant[Real, Real](taylor_three), ddf),
                    compose(square_real, affine_real(-Real.1, b)))),
                pointwise_neg(pointwise_mul(dddf, compose(cube_real, affine_real(-Real.1, b)))),
                x,
                -(pointwise_mul(constant[Real, Real](taylor_three), ddf, x) *
                    (-taylor_two * (b - x)) +
                    compose(square_real, affine_real(-Real.1, b), x) *
                        (taylor_three * dddf(x))),
                -(dddf(x) * (-taylor_three * (b - x).pow(Nat.2)) +
                    compose(cube_real, affine_real(-Real.1, b), x) * dddd(x)))
            has_derivative_at(pointwise_add(pointwise_neg(pointwise_mul(pointwise_mul(constant[Real, Real](taylor_three), ddf),
                        compose(square_real, affine_real(-Real.1, b)))),
                    pointwise_neg(pointwise_mul(dddf, compose(cube_real, affine_real(-Real.1, b))))), x,
                (-(pointwise_mul(constant[Real, Real](taylor_three), ddf, x) *
                    (-taylor_two * (b - x)) +
                    compose(square_real, affine_real(-Real.1, b), x) *
                        (taylor_three * dddf(x)))) +
                    (-(dddf(x) * (-taylor_three * (b - x).pow(Nat.2)) +
                        compose(cube_real, affine_real(-Real.1, b), x) * dddd(x))))
            taylor4_quartic_has_derivative_at(b, x)
            has_derivative_at(pointwise_mul(compose(square_real, affine_real(-Real.1, b)),
                compose(square_real, affine_real(-Real.1, b))), x,
                -taylor_four * (b - x).pow(Nat.3))
            derivative_pointwise_const_mul(taylor4_remainder_coeff(f, df, ddf, dddf, a, b),
                pointwise_mul(compose(square_real, affine_real(-Real.1, b)),
                    compose(square_real, affine_real(-Real.1, b))), x,
                -taylor_four * (b - x).pow(Nat.3))
            has_derivative_at(pointwise_mul(constant[Real, Real](taylor4_remainder_coeff(f, df, ddf, dddf, a, b)),
                pointwise_mul(compose(square_real, affine_real(-Real.1, b)),
                    compose(square_real, affine_real(-Real.1, b)))), x,
                taylor4_remainder_coeff(f, df, ddf, dddf, a, b) *
                    (-taylor_four * (b - x).pow(Nat.3)))
            derivative_pointwise_neg(pointwise_mul(constant[Real, Real](taylor4_remainder_coeff(f, df, ddf, dddf, a, b)),
                pointwise_mul(compose(square_real, affine_real(-Real.1, b)),
                    compose(square_real, affine_real(-Real.1, b)))), x,
                taylor4_remainder_coeff(f, df, ddf, dddf, a, b) *
                    (-taylor_four * (b - x).pow(Nat.3)))
            has_derivative_at(pointwise_neg(pointwise_mul(constant[Real, Real](taylor4_remainder_coeff(f, df, ddf, dddf, a, b)),
                pointwise_mul(compose(square_real, affine_real(-Real.1, b)),
                    compose(square_real, affine_real(-Real.1, b))))), x,
                -(taylor4_remainder_coeff(f, df, ddf, dddf, a, b) *
                    (-taylor_four * (b - x).pow(Nat.3))))
            derivative_pointwise_add(pointwise_add(pointwise_neg(pointwise_mul(pointwise_mul(constant[Real, Real](taylor_three), ddf),
                        compose(square_real, affine_real(-Real.1, b)))),
                    pointwise_neg(pointwise_mul(dddf, compose(cube_real, affine_real(-Real.1, b))))),
                pointwise_neg(pointwise_mul(constant[Real, Real](taylor4_remainder_coeff(f, df, ddf, dddf, a, b)),
                    pointwise_mul(compose(square_real, affine_real(-Real.1, b)),
                        compose(square_real, affine_real(-Real.1, b))))),
                x,
                (-(pointwise_mul(constant[Real, Real](taylor_three), ddf, x) *
                    (-taylor_two * (b - x)) +
                    compose(square_real, affine_real(-Real.1, b), x) *
                        (taylor_three * dddf(x)))) +
                    (-(dddf(x) * (-taylor_three * (b - x).pow(Nat.2)) +
                        compose(cube_real, affine_real(-Real.1, b), x) * dddd(x))),
                -(taylor4_remainder_coeff(f, df, ddf, dddf, a, b) *
                    (-taylor_four * (b - x).pow(Nat.3))))
            has_derivative_at(taylor4_aux_right(f, df, ddf, dddf, a, b), x,
                ((-(pointwise_mul(constant[Real, Real](taylor_three), ddf, x) *
                    (-taylor_two * (b - x)) +
                    compose(square_real, affine_real(-Real.1, b), x) *
                        (taylor_three * dddf(x)))) +
                    (-(dddf(x) * (-taylor_three * (b - x).pow(Nat.2)) +
                        compose(cube_real, affine_real(-Real.1, b), x) * dddd(x)))) +
                    -(taylor4_remainder_coeff(f, df, ddf, dddf, a, b) *
                        (-taylor_four * (b - x).pow(Nat.3))))
            taylor4_aux_pointwise(f, df, ddf, dddf, a, b) =
                pointwise_add(taylor4_aux_left(f, df, a, b), taylor4_aux_right(f, df, ddf, dddf, a, b))
            derivative_pointwise_add(taylor4_aux_left(f, df, a, b),
                taylor4_aux_right(f, df, ddf, dddf, a, b), x,
                (Real.0 + -(taylor_six * df(x))) +
                    -(pointwise_mul(constant[Real, Real](taylor_six), df, x) * -Real.1 +
                        affine_real(-Real.1, b, x) * (taylor_six * ddf(x))),
                ((-(pointwise_mul(constant[Real, Real](taylor_three), ddf, x) *
                    (-taylor_two * (b - x)) +
                    compose(square_real, affine_real(-Real.1, b), x) *
                        (taylor_three * dddf(x)))) +
                    (-(dddf(x) * (-taylor_three * (b - x).pow(Nat.2)) +
                        compose(cube_real, affine_real(-Real.1, b), x) * dddd(x)))) +
                    -(taylor4_remainder_coeff(f, df, ddf, dddf, a, b) *
                        (-taylor_four * (b - x).pow(Nat.3))))
            has_derivative_at(taylor4_aux_pointwise(f, df, ddf, dddf, a, b), x,
                ((Real.0 + -(taylor_six * df(x))) +
                    -(pointwise_mul(constant[Real, Real](taylor_six), df, x) * -Real.1 +
                        affine_real(-Real.1, b, x) * (taylor_six * ddf(x)))) +
                    (((-(pointwise_mul(constant[Real, Real](taylor_three), ddf, x) *
                        (-taylor_two * (b - x)) +
                        compose(square_real, affine_real(-Real.1, b), x) *
                            (taylor_three * dddf(x)))) +
                        (-(dddf(x) * (-taylor_three * (b - x).pow(Nat.2)) +
                            compose(cube_real, affine_real(-Real.1, b), x) * dddd(x)))) +
                        -(taylor4_remainder_coeff(f, df, ddf, dddf, a, b) *
                            (-taylor_four * (b - x).pow(Nat.3)))))
            // Simplify the rule-produced derivative value to the simplified form.
            pointwise_mul(constant[Real, Real](taylor_six), df, x) = taylor_six * df(x)
            pointwise_mul(constant[Real, Real](taylor_six), df, x) * -Real.1 =
                (taylor_six * df(x)) * -Real.1
            mul_neg_right(taylor_six * df(x), Real.1)
            (taylor_six * df(x)) * -Real.1 = -(taylor_six * df(x))
            pointwise_mul(constant[Real, Real](taylor_six), df, x) * -Real.1 =
                -(taylor_six * df(x))
            affine_real(-Real.1, b, x) = -Real.1 * x + b
            mul_neg_left(Real.1, x)
            -Real.1 * x = -(Real.1 * x)
            mul_one_left(x)
            Real.1 * x = x
            -(Real.1 * x) = -x
            -Real.1 * x = -x
            -x + b = b - x
            affine_real(-Real.1, b, x) = b - x
            pointwise_mul(constant[Real, Real](taylor_six), df, x) * -Real.1 +
                affine_real(-Real.1, b, x) * (taylor_six * ddf(x)) =
                -(taylor_six * df(x)) + (b - x) * (taylor_six * ddf(x))
            Real.0 + -(taylor_six * df(x)) = -(taylor_six * df(x))
            compose(square_real, affine_real(-Real.1, b), x) =
                square_real(affine_real(-Real.1, b, x))
            square_real(affine_real(-Real.1, b, x)) = square_real(b - x)
            real_pow_two_eq_mul(b - x)
            (b - x).pow(Nat.2) = (b - x) * (b - x)
            square_real(b - x) = (b - x).pow(Nat.2)
            compose(square_real, affine_real(-Real.1, b), x) = (b - x).pow(Nat.2)
            compose(cube_real, affine_real(-Real.1, b), x) =
                cube_real(affine_real(-Real.1, b, x))
            cube_real(affine_real(-Real.1, b, x)) = cube_real(b - x)
            real_pow_three_eq_mul(b - x)
            (b - x).pow(Nat.3) = (b - x) * (b - x) * (b - x)
            cube_real(b - x) = (b - x).pow(Nat.3)
            compose(cube_real, affine_real(-Real.1, b), x) = (b - x).pow(Nat.3)
            pointwise_mul(constant[Real, Real](taylor_three), ddf, x) = taylor_three * ddf(x)
            mul_neg_right(taylor_three * ddf(x), b - x)
            (taylor_three * ddf(x)) * (-(b - x)) = -(taylor_three * ddf(x) * (b - x))
            pointwise_mul(constant[Real, Real](taylor_three), ddf, x) * (-taylor_two * (b - x)) =
                taylor_three * ddf(x) * (-taylor_two * (b - x))
            taylor_three * ddf(x) * (-taylor_two * (b - x)) =
                -(taylor_three * ddf(x) * (taylor_two * (b - x)))
            pointwise_mul(constant[Real, Real](taylor_three), ddf, x) * (-taylor_two * (b - x)) =
                -(taylor_three * ddf(x) * (taylor_two * (b - x)))
            mul_assoc(taylor_three, ddf(x), taylor_two * (b - x))
            taylor_three * ddf(x) * (taylor_two * (b - x)) =
                taylor_three * (ddf(x) * (taylor_two * (b - x)))
            mul_assoc(ddf(x), taylor_two, b - x)
            ddf(x) * (taylor_two * (b - x)) = (ddf(x) * taylor_two) * (b - x)
            real_mul_comm(ddf(x), taylor_two)
            ddf(x) * taylor_two = taylor_two * ddf(x)
            (ddf(x) * taylor_two) * (b - x) = (taylor_two * ddf(x)) * (b - x)
            ddf(x) * (taylor_two * (b - x)) = (taylor_two * ddf(x)) * (b - x)
            taylor_three * (ddf(x) * (taylor_two * (b - x))) =
                taylor_three * ((taylor_two * ddf(x)) * (b - x))
            taylor_three * ((taylor_two * ddf(x)) * (b - x)) =
                taylor_three * (taylor_two * ddf(x)) * (b - x)
            mul_assoc(taylor_three, taylor_two * ddf(x), b - x)
            taylor_three * ((taylor_two * ddf(x)) * (b - x)) =
                (taylor_three * (taylor_two * ddf(x))) * (b - x)
            mul_assoc(taylor_three, taylor_two, ddf(x))
            taylor_three * (taylor_two * ddf(x)) = (taylor_three * taylor_two) * ddf(x)
            taylor_six = taylor_two * taylor_three
            real_mul_comm(taylor_two, taylor_three)
            taylor_two * taylor_three = taylor_three * taylor_two
            taylor_six = taylor_three * taylor_two
            (taylor_three * taylor_two) * ddf(x) = taylor_six * ddf(x)
            taylor_three * (taylor_two * ddf(x)) = taylor_six * ddf(x)
            taylor_three * ((taylor_two * ddf(x)) * (b - x)) =
                (taylor_six * ddf(x)) * (b - x)
            mul_assoc(taylor_six, ddf(x), b - x)
            (taylor_six * ddf(x)) * (b - x) = taylor_six * (ddf(x) * (b - x))
            real_mul_comm(ddf(x), b - x)
            ddf(x) * (b - x) = (b - x) * ddf(x)
            taylor_six * (ddf(x) * (b - x)) = taylor_six * ((b - x) * ddf(x))
            (taylor_six * ddf(x)) * (b - x) = taylor_six * ((b - x) * ddf(x))
            taylor_three * ((taylor_two * ddf(x)) * (b - x)) =
                taylor_six * ((b - x) * ddf(x))
            taylor_three * ddf(x) * (taylor_two * (b - x)) =
                taylor_six * ((b - x) * ddf(x))
            mul_assoc(taylor_six, b - x, ddf(x))
            taylor_six * ((b - x) * ddf(x)) = (taylor_six * (b - x)) * ddf(x)
            real_mul_comm(taylor_six, b - x)
            taylor_six * (b - x) = (b - x) * taylor_six
            (taylor_six * (b - x)) * ddf(x) = ((b - x) * taylor_six) * ddf(x)
            mul_assoc(b - x, taylor_six, ddf(x))
            ((b - x) * taylor_six) * ddf(x) = (b - x) * (taylor_six * ddf(x))
            (taylor_six * (b - x)) * ddf(x) = (b - x) * (taylor_six * ddf(x))
            taylor_six * ((b - x) * ddf(x)) = (b - x) * (taylor_six * ddf(x))
            taylor_three * ddf(x) * (taylor_two * (b - x)) =
                (b - x) * (taylor_six * ddf(x))
            pointwise_mul(constant[Real, Real](taylor_three), ddf, x) * (-taylor_two * (b - x)) =
                -((b - x) * (taylor_six * ddf(x)))
            pointwise_mul(constant[Real, Real](taylor_three), ddf, x) * (-taylor_two * (b - x)) +
                compose(square_real, affine_real(-Real.1, b), x) * (taylor_three * dddf(x)) =
                -((b - x) * (taylor_six * ddf(x))) + (b - x).pow(Nat.2) * (taylor_three * dddf(x))
            -(pointwise_mul(constant[Real, Real](taylor_three), ddf, x) * (-taylor_two * (b - x)) +
                compose(square_real, affine_real(-Real.1, b), x) * (taylor_three * dddf(x))) =
                -(-((b - x) * (taylor_six * ddf(x))) + (b - x).pow(Nat.2) * (taylor_three * dddf(x)))
            neg_distrib(-((b - x) * (taylor_six * ddf(x))), (b - x).pow(Nat.2) * (taylor_three * dddf(x)))
            -(-((b - x) * (taylor_six * ddf(x))) + (b - x).pow(Nat.2) * (taylor_three * dddf(x))) =
                -(-((b - x) * (taylor_six * ddf(x)))) + -((b - x).pow(Nat.2) * (taylor_three * dddf(x)))
            neg_neg((b - x) * (taylor_six * ddf(x)))
            -(-((b - x) * (taylor_six * ddf(x)))) = (b - x) * (taylor_six * ddf(x))
            -(-((b - x) * (taylor_six * ddf(x)))) + -((b - x).pow(Nat.2) * (taylor_three * dddf(x))) =
                (b - x) * (taylor_six * ddf(x)) + -((b - x).pow(Nat.2) * (taylor_three * dddf(x)))
            (b - x) * (taylor_six * ddf(x)) + -((b - x).pow(Nat.2) * (taylor_three * dddf(x))) =
                (b - x) * (taylor_six * ddf(x)) - (b - x).pow(Nat.2) * (taylor_three * dddf(x))
            -(-((b - x) * (taylor_six * ddf(x)))) + -((b - x).pow(Nat.2) * (taylor_three * dddf(x))) =
                (b - x) * (taylor_six * ddf(x)) - (b - x).pow(Nat.2) * (taylor_three * dddf(x))
            -(pointwise_mul(constant[Real, Real](taylor_three), ddf, x) * (-taylor_two * (b - x)) +
                compose(square_real, affine_real(-Real.1, b), x) * (taylor_three * dddf(x))) =
                (b - x) * (taylor_six * ddf(x)) - (b - x).pow(Nat.2) * (taylor_three * dddf(x))
            pointwise_mul(constant[Real, Real](taylor_six), df, x) * -Real.1 +
                affine_real(-Real.1, b, x) * (taylor_six * ddf(x)) =
                -(taylor_six * df(x)) + (b - x) * (taylor_six * ddf(x))
            -(pointwise_mul(constant[Real, Real](taylor_six), df, x) * -Real.1 +
                affine_real(-Real.1, b, x) * (taylor_six * ddf(x))) =
                -(-(taylor_six * df(x)) + (b - x) * (taylor_six * ddf(x)))
            neg_distrib(-(taylor_six * df(x)), (b - x) * (taylor_six * ddf(x)))
            -(-(taylor_six * df(x)) + (b - x) * (taylor_six * ddf(x))) =
                -(-(taylor_six * df(x))) + -((b - x) * (taylor_six * ddf(x)))
            neg_neg(taylor_six * df(x))
            -(-(taylor_six * df(x))) = taylor_six * df(x)
            -(-(taylor_six * df(x))) + -((b - x) * (taylor_six * ddf(x))) =
                taylor_six * df(x) + -((b - x) * (taylor_six * ddf(x)))
            taylor_six * df(x) + -((b - x) * (taylor_six * ddf(x))) =
                taylor_six * df(x) - (b - x) * (taylor_six * ddf(x))
            -(pointwise_mul(constant[Real, Real](taylor_six), df, x) * -Real.1 +
                affine_real(-Real.1, b, x) * (taylor_six * ddf(x))) =
                taylor_six * df(x) - (b - x) * (taylor_six * ddf(x))
            (Real.0 + -(taylor_six * df(x))) +
                -(pointwise_mul(constant[Real, Real](taylor_six), df, x) * -Real.1 +
                    affine_real(-Real.1, b, x) * (taylor_six * ddf(x))) =
                -(taylor_six * df(x)) + (taylor_six * df(x) - (b - x) * (taylor_six * ddf(x)))
            add_assoc(-(taylor_six * df(x)), taylor_six * df(x),
                -((b - x) * (taylor_six * ddf(x))))
            -(taylor_six * df(x)) + (taylor_six * df(x) - (b - x) * (taylor_six * ddf(x))) =
                -(taylor_six * df(x)) + taylor_six * df(x) - (b - x) * (taylor_six * ddf(x))
            (Real.0 + -(taylor_six * df(x))) +
                -(pointwise_mul(constant[Real, Real](taylor_six), df, x) * -Real.1 +
                    affine_real(-Real.1, b, x) * (taylor_six * ddf(x))) =
                -(taylor_six * df(x)) + taylor_six * df(x) - (b - x) * (taylor_six * ddf(x))
            -(taylor_six * df(x)) + taylor_six * df(x) = Real.0
            -(taylor_six * df(x)) + taylor_six * df(x) - (b - x) * (taylor_six * ddf(x)) =
                Real.0 - (b - x) * (taylor_six * ddf(x))
            Real.0 - (b - x) * (taylor_six * ddf(x)) =
                -((b - x) * (taylor_six * ddf(x)))
            -(taylor_six * df(x)) + taylor_six * df(x) - (b - x) * (taylor_six * ddf(x)) =
                -((b - x) * (taylor_six * ddf(x)))
            (Real.0 + -(taylor_six * df(x))) +
                -(pointwise_mul(constant[Real, Real](taylor_six), df, x) * -Real.1 +
                    affine_real(-Real.1, b, x) * (taylor_six * ddf(x))) =
                -((b - x) * (taylor_six * ddf(x)))
            ((Real.0 + -(taylor_six * df(x))) +
                -(pointwise_mul(constant[Real, Real](taylor_six), df, x) * -Real.1 +
                    affine_real(-Real.1, b, x) * (taylor_six * ddf(x)))) +
                (-(pointwise_mul(constant[Real, Real](taylor_three), ddf, x) * (-taylor_two * (b - x)) +
                    compose(square_real, affine_real(-Real.1, b), x) * (taylor_three * dddf(x)))) =
                -((b - x) * (taylor_six * ddf(x))) +
                    ((b - x) * (taylor_six * ddf(x)) - (b - x).pow(Nat.2) * (taylor_three * dddf(x)))
            add_assoc(-((b - x) * (taylor_six * ddf(x))),
                (b - x) * (taylor_six * ddf(x)), -((b - x).pow(Nat.2) * (taylor_three * dddf(x))))
            -((b - x) * (taylor_six * ddf(x))) +
                ((b - x) * (taylor_six * ddf(x)) + -((b - x).pow(Nat.2) * (taylor_three * dddf(x)))) =
                (-((b - x) * (taylor_six * ddf(x))) + (b - x) * (taylor_six * ddf(x))) +
                    -((b - x).pow(Nat.2) * (taylor_three * dddf(x)))
            -((b - x) * (taylor_six * ddf(x))) + (b - x) * (taylor_six * ddf(x)) = Real.0
            (-((b - x) * (taylor_six * ddf(x))) + (b - x) * (taylor_six * ddf(x))) +
                -((b - x).pow(Nat.2) * (taylor_three * dddf(x))) =
                Real.0 + -((b - x).pow(Nat.2) * (taylor_three * dddf(x)))
            add_zero_left(-((b - x).pow(Nat.2) * (taylor_three * dddf(x))))
            Real.0 + -((b - x).pow(Nat.2) * (taylor_three * dddf(x))) =
                -((b - x).pow(Nat.2) * (taylor_three * dddf(x)))
            (-((b - x) * (taylor_six * ddf(x))) + (b - x) * (taylor_six * ddf(x))) +
                -((b - x).pow(Nat.2) * (taylor_three * dddf(x))) =
                -((b - x).pow(Nat.2) * (taylor_three * dddf(x)))
            -((b - x) * (taylor_six * ddf(x))) +
                ((b - x) * (taylor_six * ddf(x)) - (b - x).pow(Nat.2) * (taylor_three * dddf(x))) =
                -((b - x).pow(Nat.2) * (taylor_three * dddf(x)))
            // Now the dddf * g3 piece.
            dddf(x) * (-taylor_three * (b - x).pow(Nat.2)) =
                -(dddf(x) * (taylor_three * (b - x).pow(Nat.2)))
            -(dddf(x) * (-taylor_three * (b - x).pow(Nat.2)) +
                compose(cube_real, affine_real(-Real.1, b), x) * dddd(x)) =
                -(dddf(x) * (-taylor_three * (b - x).pow(Nat.2)) +
                    (b - x).pow(Nat.3) * dddd(x))
            neg_distrib(dddf(x) * (-taylor_three * (b - x).pow(Nat.2)),
                (b - x).pow(Nat.3) * dddd(x))
            -(dddf(x) * (-taylor_three * (b - x).pow(Nat.2)) +
                (b - x).pow(Nat.3) * dddd(x)) =
                -(dddf(x) * (-taylor_three * (b - x).pow(Nat.2))) + -((b - x).pow(Nat.3) * dddd(x))
            mul_neg_right(dddf(x), taylor_three * (b - x).pow(Nat.2))
            dddf(x) * (-(taylor_three * (b - x).pow(Nat.2))) =
                -(dddf(x) * (taylor_three * (b - x).pow(Nat.2)))
            dddf(x) * (-taylor_three * (b - x).pow(Nat.2)) =
                dddf(x) * (-(taylor_three * (b - x).pow(Nat.2)))
            dddf(x) * (-taylor_three * (b - x).pow(Nat.2)) =
                -(dddf(x) * (taylor_three * (b - x).pow(Nat.2)))
            neg_neg(dddf(x) * (taylor_three * (b - x).pow(Nat.2)))
            -(-(dddf(x) * (taylor_three * (b - x).pow(Nat.2)))) =
                dddf(x) * (taylor_three * (b - x).pow(Nat.2))
            -(dddf(x) * (-taylor_three * (b - x).pow(Nat.2))) + -((b - x).pow(Nat.3) * dddd(x)) =
                dddf(x) * (taylor_three * (b - x).pow(Nat.2)) + -((b - x).pow(Nat.3) * dddd(x))
            dddf(x) * (taylor_three * (b - x).pow(Nat.2)) + -((b - x).pow(Nat.3) * dddd(x)) =
                dddf(x) * (taylor_three * (b - x).pow(Nat.2)) - (b - x).pow(Nat.3) * dddd(x)
            -(dddf(x) * (-taylor_three * (b - x).pow(Nat.2)) +
                (b - x).pow(Nat.3) * dddd(x)) =
                dddf(x) * (taylor_three * (b - x).pow(Nat.2)) - (b - x).pow(Nat.3) * dddd(x)
            // Combine: the quadratic pieces cancel.
            (-(pointwise_mul(constant[Real, Real](taylor_three), ddf, x) * (-taylor_two * (b - x)) +
                compose(square_real, affine_real(-Real.1, b), x) * (taylor_three * dddf(x)))) +
                (-(dddf(x) * (-taylor_three * (b - x).pow(Nat.2)) +
                    compose(cube_real, affine_real(-Real.1, b), x) * dddd(x))) =
                ((b - x) * (taylor_six * ddf(x)) - (b - x).pow(Nat.2) * (taylor_three * dddf(x))) +
                    (dddf(x) * (taylor_three * (b - x).pow(Nat.2)) - (b - x).pow(Nat.3) * dddd(x))
            real_mul_comm(dddf(x), taylor_three)
            dddf(x) * taylor_three = taylor_three * dddf(x)
            mul_assoc(dddf(x), taylor_three, (b - x).pow(Nat.2))
            dddf(x) * (taylor_three * (b - x).pow(Nat.2)) =
                (dddf(x) * taylor_three) * (b - x).pow(Nat.2)
            (dddf(x) * taylor_three) * (b - x).pow(Nat.2) =
                (taylor_three * dddf(x)) * (b - x).pow(Nat.2)
            dddf(x) * (taylor_three * (b - x).pow(Nat.2)) =
                (taylor_three * dddf(x)) * (b - x).pow(Nat.2)
            real_mul_comm(taylor_three * dddf(x), (b - x).pow(Nat.2))
            (taylor_three * dddf(x)) * (b - x).pow(Nat.2) =
                (b - x).pow(Nat.2) * (taylor_three * dddf(x))
            dddf(x) * (taylor_three * (b - x).pow(Nat.2)) =
                (b - x).pow(Nat.2) * (taylor_three * dddf(x))
            ((b - x) * (taylor_six * ddf(x)) - (b - x).pow(Nat.2) * (taylor_three * dddf(x))) +
                (dddf(x) * (taylor_three * (b - x).pow(Nat.2)) - (b - x).pow(Nat.3) * dddd(x)) =
                (b - x) * (taylor_six * ddf(x)) - (b - x).pow(Nat.3) * dddd(x)
            (-(pointwise_mul(constant[Real, Real](taylor_three), ddf, x) * (-taylor_two * (b - x)) +
                compose(square_real, affine_real(-Real.1, b), x) * (taylor_three * dddf(x)))) +
                (-(dddf(x) * (-taylor_three * (b - x).pow(Nat.2)) +
                    compose(cube_real, affine_real(-Real.1, b), x) * dddd(x))) =
                (b - x) * (taylor_six * ddf(x)) - (b - x).pow(Nat.3) * dddd(x)
            // The quartic piece's negation.
            mul_neg_right(taylor4_remainder_coeff(f, df, ddf, dddf, a, b),
                taylor_four * (b - x).pow(Nat.3))
            taylor4_remainder_coeff(f, df, ddf, dddf, a, b) * (-(taylor_four * (b - x).pow(Nat.3))) =
                -(taylor4_remainder_coeff(f, df, ddf, dddf, a, b) * (taylor_four * (b - x).pow(Nat.3)))
            taylor4_remainder_coeff(f, df, ddf, dddf, a, b) * (-taylor_four * (b - x).pow(Nat.3)) =
                taylor4_remainder_coeff(f, df, ddf, dddf, a, b) * (-(taylor_four * (b - x).pow(Nat.3)))
            taylor4_remainder_coeff(f, df, ddf, dddf, a, b) * (-taylor_four * (b - x).pow(Nat.3)) =
                -(taylor4_remainder_coeff(f, df, ddf, dddf, a, b) * (taylor_four * (b - x).pow(Nat.3)))
            neg_neg(taylor4_remainder_coeff(f, df, ddf, dddf, a, b) * (taylor_four * (b - x).pow(Nat.3)))
            -(-(taylor4_remainder_coeff(f, df, ddf, dddf, a, b) * (taylor_four * (b - x).pow(Nat.3)))) =
                taylor4_remainder_coeff(f, df, ddf, dddf, a, b) * (taylor_four * (b - x).pow(Nat.3))
            -(taylor4_remainder_coeff(f, df, ddf, dddf, a, b) * (-taylor_four * (b - x).pow(Nat.3))) =
                taylor4_remainder_coeff(f, df, ddf, dddf, a, b) * (taylor_four * (b - x).pow(Nat.3))
            mul_assoc(taylor4_remainder_coeff(f, df, ddf, dddf, a, b), taylor_four, (b - x).pow(Nat.3))
            taylor4_remainder_coeff(f, df, ddf, dddf, a, b) * (taylor_four * (b - x).pow(Nat.3)) =
                (taylor4_remainder_coeff(f, df, ddf, dddf, a, b) * taylor_four) * (b - x).pow(Nat.3)
            real_mul_comm(taylor4_remainder_coeff(f, df, ddf, dddf, a, b), taylor_four)
            taylor4_remainder_coeff(f, df, ddf, dddf, a, b) * taylor_four =
                taylor_four * taylor4_remainder_coeff(f, df, ddf, dddf, a, b)
            (taylor4_remainder_coeff(f, df, ddf, dddf, a, b) * taylor_four) * (b - x).pow(Nat.3) =
                (taylor_four * taylor4_remainder_coeff(f, df, ddf, dddf, a, b)) * (b - x).pow(Nat.3)
            taylor4_remainder_coeff(f, df, ddf, dddf, a, b) * (taylor_four * (b - x).pow(Nat.3)) =
                (taylor_four * taylor4_remainder_coeff(f, df, ddf, dddf, a, b)) * (b - x).pow(Nat.3)
            -(taylor4_remainder_coeff(f, df, ddf, dddf, a, b) * (-taylor_four * (b - x).pow(Nat.3))) =
                (taylor_four * taylor4_remainder_coeff(f, df, ddf, dddf, a, b)) * (b - x).pow(Nat.3)
            // Assemble the full derivative.
            ((-(pointwise_mul(constant[Real, Real](taylor_three), ddf, x) * (-taylor_two * (b - x)) +
                compose(square_real, affine_real(-Real.1, b), x) * (taylor_three * dddf(x)))) +
                (-(dddf(x) * (-taylor_three * (b - x).pow(Nat.2)) +
                    compose(cube_real, affine_real(-Real.1, b), x) * dddd(x)))) +
                -(taylor4_remainder_coeff(f, df, ddf, dddf, a, b) * (-taylor_four * (b - x).pow(Nat.3))) =
                (b - x) * (taylor_six * ddf(x)) - (b - x).pow(Nat.3) * dddd(x) +
                    (taylor_four * taylor4_remainder_coeff(f, df, ddf, dddf, a, b)) * (b - x).pow(Nat.3)
            // The whole pointwise derivative.
            ((Real.0 + -(taylor_six * df(x))) +
                -(pointwise_mul(constant[Real, Real](taylor_six), df, x) * -Real.1 +
                    affine_real(-Real.1, b, x) * (taylor_six * ddf(x)))) +
                (((-(pointwise_mul(constant[Real, Real](taylor_three), ddf, x) * (-taylor_two * (b - x)) +
                    compose(square_real, affine_real(-Real.1, b), x) * (taylor_three * dddf(x)))) +
                    (-(dddf(x) * (-taylor_three * (b - x).pow(Nat.2)) +
                        compose(cube_real, affine_real(-Real.1, b), x) * dddd(x)))) +
                    -(taylor4_remainder_coeff(f, df, ddf, dddf, a, b) * (-taylor_four * (b - x).pow(Nat.3)))) =
                -((b - x) * (taylor_six * ddf(x))) +
                    ((b - x) * (taylor_six * ddf(x)) - (b - x).pow(Nat.3) * dddd(x) +
                        (taylor_four * taylor4_remainder_coeff(f, df, ddf, dddf, a, b)) * (b - x).pow(Nat.3))
            taylor4_neg_cancel((b - x) * (taylor_six * ddf(x)), (b - x).pow(Nat.3) * dddd(x))
            -((b - x) * (taylor_six * ddf(x))) +
                ((b - x) * (taylor_six * ddf(x)) - (b - x).pow(Nat.3) * dddd(x)) =
                -((b - x).pow(Nat.3) * dddd(x))
            add_assoc(-((b - x) * (taylor_six * ddf(x))),
                (b - x) * (taylor_six * ddf(x)) - (b - x).pow(Nat.3) * dddd(x),
                (taylor_four * taylor4_remainder_coeff(f, df, ddf, dddf, a, b)) * (b - x).pow(Nat.3))
            -((b - x) * (taylor_six * ddf(x))) +
                ((b - x) * (taylor_six * ddf(x)) - (b - x).pow(Nat.3) * dddd(x) +
                    (taylor_four * taylor4_remainder_coeff(f, df, ddf, dddf, a, b)) * (b - x).pow(Nat.3)) =
                (-((b - x) * (taylor_six * ddf(x))) +
                    ((b - x) * (taylor_six * ddf(x)) - (b - x).pow(Nat.3) * dddd(x))) +
                    (taylor_four * taylor4_remainder_coeff(f, df, ddf, dddf, a, b)) * (b - x).pow(Nat.3)
            (-((b - x) * (taylor_six * ddf(x))) +
                ((b - x) * (taylor_six * ddf(x)) - (b - x).pow(Nat.3) * dddd(x))) +
                (taylor_four * taylor4_remainder_coeff(f, df, ddf, dddf, a, b)) * (b - x).pow(Nat.3) =
                -((b - x).pow(Nat.3) * dddd(x)) +
                    (taylor_four * taylor4_remainder_coeff(f, df, ddf, dddf, a, b)) * (b - x).pow(Nat.3)
            -((b - x) * (taylor_six * ddf(x))) +
                ((b - x) * (taylor_six * ddf(x)) - (b - x).pow(Nat.3) * dddd(x) +
                    (taylor_four * taylor4_remainder_coeff(f, df, ddf, dddf, a, b)) * (b - x).pow(Nat.3)) =
                -((b - x).pow(Nat.3) * dddd(x)) +
                    (taylor_four * taylor4_remainder_coeff(f, df, ddf, dddf, a, b)) * (b - x).pow(Nat.3)
            real_mul_comm((b - x).pow(Nat.3), taylor_four * taylor4_remainder_coeff(f, df, ddf, dddf, a, b))
            (taylor_four * taylor4_remainder_coeff(f, df, ddf, dddf, a, b)) * (b - x).pow(Nat.3) =
                (b - x).pow(Nat.3) * (taylor_four * taylor4_remainder_coeff(f, df, ddf, dddf, a, b))
            -((b - x).pow(Nat.3) * dddd(x)) +
                (taylor_four * taylor4_remainder_coeff(f, df, ddf, dddf, a, b)) * (b - x).pow(Nat.3) =
                -((b - x).pow(Nat.3) * dddd(x)) +
                    (b - x).pow(Nat.3) * (taylor_four * taylor4_remainder_coeff(f, df, ddf, dddf, a, b))
            add_comm(-((b - x).pow(Nat.3) * dddd(x)),
                (b - x).pow(Nat.3) * (taylor_four * taylor4_remainder_coeff(f, df, ddf, dddf, a, b)))
            -((b - x).pow(Nat.3) * dddd(x)) +
                (b - x).pow(Nat.3) * (taylor_four * taylor4_remainder_coeff(f, df, ddf, dddf, a, b)) =
                (b - x).pow(Nat.3) * (taylor_four * taylor4_remainder_coeff(f, df, ddf, dddf, a, b)) -
                    (b - x).pow(Nat.3) * dddd(x)
            mul_distrib_right((b - x).pow(Nat.3),
                taylor_four * taylor4_remainder_coeff(f, df, ddf, dddf, a, b), -dddd(x))
            (b - x).pow(Nat.3) * ((taylor_four * taylor4_remainder_coeff(f, df, ddf, dddf, a, b)) + -dddd(x)) =
                (b - x).pow(Nat.3) * (taylor_four * taylor4_remainder_coeff(f, df, ddf, dddf, a, b)) +
                    (b - x).pow(Nat.3) * (-dddd(x))
            mul_neg_right((b - x).pow(Nat.3), dddd(x))
            (b - x).pow(Nat.3) * (-dddd(x)) = -((b - x).pow(Nat.3) * dddd(x))
            (b - x).pow(Nat.3) * ((taylor_four * taylor4_remainder_coeff(f, df, ddf, dddf, a, b)) + -dddd(x)) =
                (b - x).pow(Nat.3) * (taylor_four * taylor4_remainder_coeff(f, df, ddf, dddf, a, b)) -
                    (b - x).pow(Nat.3) * dddd(x)
            (b - x).pow(Nat.3) * (taylor_four * taylor4_remainder_coeff(f, df, ddf, dddf, a, b) - dddd(x)) =
                (b - x).pow(Nat.3) * (taylor_four * taylor4_remainder_coeff(f, df, ddf, dddf, a, b)) -
                    (b - x).pow(Nat.3) * dddd(x)
            ((Real.0 + -(taylor_six * df(x))) +
                -(pointwise_mul(constant[Real, Real](taylor_six), df, x) * -Real.1 +
                    affine_real(-Real.1, b, x) * (taylor_six * ddf(x)))) +
                (((-(pointwise_mul(constant[Real, Real](taylor_three), ddf, x) * (-taylor_two * (b - x)) +
                    compose(square_real, affine_real(-Real.1, b), x) * (taylor_three * dddf(x)))) +
                    (-(dddf(x) * (-taylor_three * (b - x).pow(Nat.2)) +
                        compose(cube_real, affine_real(-Real.1, b), x) * dddd(x)))) +
                    -(taylor4_remainder_coeff(f, df, ddf, dddf, a, b) * (-taylor_four * (b - x).pow(Nat.3)))) =
                (b - x).pow(Nat.3) * (taylor_four * taylor4_remainder_coeff(f, df, ddf, dddf, a, b) - dddd(x))
            taylor4_aux_derivative(f, df, ddf, dddf, dddd, a, b, x) =
                (b - x).pow(Nat.3) * (taylor_four * taylor4_remainder_coeff(f, df, ddf, dddf, a, b) - dddd(x))
            ((Real.0 + -(taylor_six * df(x))) +
                -(pointwise_mul(constant[Real, Real](taylor_six), df, x) * -Real.1 +
                    affine_real(-Real.1, b, x) * (taylor_six * ddf(x)))) +
                (((-(pointwise_mul(constant[Real, Real](taylor_three), ddf, x) * (-taylor_two * (b - x)) +
                    compose(square_real, affine_real(-Real.1, b), x) * (taylor_three * dddf(x)))) +
                    (-(dddf(x) * (-taylor_three * (b - x).pow(Nat.2)) +
                        compose(cube_real, affine_real(-Real.1, b), x) * dddd(x)))) +
                    -(taylor4_remainder_coeff(f, df, ddf, dddf, a, b) * (-taylor_four * (b - x).pow(Nat.3)))) =
                taylor4_aux_derivative(f, df, ddf, dddf, dddd, a, b, x)
            has_derivative_at(taylor4_aux_pointwise(f, df, ddf, dddf, a, b), x,
                taylor4_aux_derivative(f, df, ddf, dddf, dddd, a, b, x))
            taylor4_aux_eq_pointwise(f, df, ddf, dddf, a, b)
            taylor4_aux(f, df, ddf, dddf, a, b) = taylor4_aux_pointwise(f, df, ddf, dddf, a, b)
            function_eq_transport_predicate_rev(
                function(h: Real -> Real) {
                    has_derivative_at(h, x, taylor4_aux_derivative(f, df, ddf, dddf, dddd, a, b, x))
                },
                taylor4_aux(f, df, ddf, dddf, a, b),
                taylor4_aux_pointwise(f, df, ddf, dddf, a, b))
            has_derivative_at(taylor4_aux(f, df, ddf, dddf, a, b), x,
                taylor4_aux_derivative(f, df, ddf, dddf, dddd, a, b, x))
        }
        is_derivative_fn_iff(taylor4_aux(f, df, ddf, dddf, a, b),
            taylor4_aux_derivative(f, df, ddf, dddf, dddd, a, b))
        is_derivative_fn(taylor4_aux(f, df, ddf, dddf, a, b),
            taylor4_aux_derivative(f, df, ddf, dddf, dddd, a, b))
    }
}

/// Subtracting a sum of three equals subtracting each term: a - (b + c + d) = a - b - c - d.
theorem taylor4_sub_sum(a: Real, b: Real, c: Real, d: Real) {
    a - (b + c + d) = a - b - c - d
} by {
    taylor3_sub_sum(a, b, c + d)
    a - (b + (c + d)) = a - b - (c + d)
    add_assoc(b, c, d)
    (b + c) + d = b + c + d
    b + (c + d) = b + c + d
    a - (b + c + d) = a - b - (c + d)
    taylor3_sub_sum(a - b, c, d)
    (a - b) - (c + d) = (a - b) - c - d
    a - b - c - d = (a - b) - c - d
    a - b - (c + d) = a - b - c - d
    a - (b + c + d) = a - b - c - d
}

/// Adding the subtracted terms back: (a - b - c - d - e) + b + c + d + e = a.
theorem taylor4_sub_add_cancel4(a: Real, b: Real, c: Real, d: Real, e: Real) {
    (a - b - c - d - e) + b + c + d + e = a
} by {
    taylor4_sub_sum(a - b, c, d, e)
    (a - b) - (c + d + e) = (a - b) - c - d - e
    a - b - c - d - e = (a - b) - c - d - e
    taylor2_sub_sub_add(a, b, c + d + e)
    (a - b - (c + d + e)) + b = a - (c + d + e)
    a - b - (c + d + e) = a - b - c - d - e
    (a - b - c - d - e) + b = a - (c + d + e)
    taylor4_sub_sum(a, c, d, e)
    a - (c + d + e) = a - c - d - e
    (a - b - c - d - e) + b = a - c - d - e
    taylor3_sub_add_cancel3(a, c, d, e)
    (a - c - d - e) + c + d + e = a
    ((a - b - c - d - e) + b) + c + d + e = a
    add_assoc(a - b - c - d - e, b, c + d + e)
    (a - b - c - d - e) + (b + c + d + e) = (a - b - c - d - e) + b + c + d + e
    add_assoc(c, d, e)
    (c + d) + e = c + d + e
    add_assoc(b, c + d, e)
    b + ((c + d) + e) = (b + (c + d)) + e
    b + (c + d + e) = (b + (c + d)) + e
    (b + (c + d)) + e = b + c + d + e
    b + (c + d + e) = b + c + d + e
    (a - b - c - d - e) + (b + c + d + e) = (a - b - c - d - e) + (b + (c + d + e))
    add_assoc(a - b - c - d - e, b, c + d + e)
    (a - b - c - d - e) + (b + (c + d + e)) = ((a - b - c - d - e) + b) + (c + d + e)
    (a - b - c - d - e) + (b + c + d + e) = ((a - b - c - d - e) + b) + (c + d + e)
    ((a - b - c - d - e) + b) + (c + d + e) = ((a - b - c - d - e) + b) + c + d + e
    (a - b - c - d - e) + (b + c + d + e) = ((a - b - c - d - e) + b) + c + d + e
    (a - b - c - d - e) + b + c + d + e = ((a - b - c - d - e) + b) + c + d + e
    (a - b - c - d - e) + b + c + d + e = a
}

/// Dividing a sixfold difference chain by six recovers the difference chain.
theorem taylor4_div_six_chain(a: Real, b: Real, c: Real, d: Real, e: Real) {
    (taylor_six * a - taylor_six * b - taylor_six * c - taylor_three * d - e) / taylor_six =
        a - b - c - d / taylor_two - e / taylor_six
} by {
    taylor_six_not_zero
    taylor_six != Real.0
    (taylor_six * a - taylor_six * b - taylor_six * c - taylor_three * d - e) / taylor_six =
        (taylor_six * a - taylor_six * b - taylor_six * c - taylor_three * d - e) * taylor_six.inverse
    taylor3_sub_mul(taylor_six * a - taylor_six * b - taylor_six * c - taylor_three * d, e, taylor_six.inverse)
    ((taylor_six * a - taylor_six * b - taylor_six * c - taylor_three * d) - e) * taylor_six.inverse =
        (taylor_six * a - taylor_six * b - taylor_six * c - taylor_three * d) * taylor_six.inverse -
            e * taylor_six.inverse
    taylor_six * a - taylor_six * b - taylor_six * c - taylor_three * d - e =
        (taylor_six * a - taylor_six * b - taylor_six * c - taylor_three * d) - e
    (taylor_six * a - taylor_six * b - taylor_six * c - taylor_three * d - e) * taylor_six.inverse =
        (taylor_six * a - taylor_six * b - taylor_six * c - taylor_three * d) * taylor_six.inverse -
            e * taylor_six.inverse
    taylor3_sub_mul(taylor_six * a - taylor_six * b - taylor_six * c, taylor_three * d, taylor_six.inverse)
    ((taylor_six * a - taylor_six * b - taylor_six * c) - taylor_three * d) * taylor_six.inverse =
        (taylor_six * a - taylor_six * b - taylor_six * c) * taylor_six.inverse -
            (taylor_three * d) * taylor_six.inverse
    taylor_six * a - taylor_six * b - taylor_six * c - taylor_three * d =
        (taylor_six * a - taylor_six * b - taylor_six * c) - taylor_three * d
    (taylor_six * a - taylor_six * b - taylor_six * c - taylor_three * d) * taylor_six.inverse =
        (taylor_six * a - taylor_six * b - taylor_six * c) * taylor_six.inverse -
            (taylor_three * d) * taylor_six.inverse
    taylor3_sub_mul(taylor_six * a - taylor_six * b, taylor_six * c, taylor_six.inverse)
    ((taylor_six * a - taylor_six * b) - taylor_six * c) * taylor_six.inverse =
        (taylor_six * a - taylor_six * b) * taylor_six.inverse -
            (taylor_six * c) * taylor_six.inverse
    taylor_six * a - taylor_six * b - taylor_six * c =
        (taylor_six * a - taylor_six * b) - taylor_six * c
    (taylor_six * a - taylor_six * b - taylor_six * c) * taylor_six.inverse =
        (taylor_six * a - taylor_six * b) * taylor_six.inverse -
            (taylor_six * c) * taylor_six.inverse
    taylor3_sub_mul(taylor_six * a, taylor_six * b, taylor_six.inverse)
    (taylor_six * a - taylor_six * b) * taylor_six.inverse =
        (taylor_six * a) * taylor_six.inverse - (taylor_six * b) * taylor_six.inverse
    real_mul_inverse_comm(taylor_six, a)
    (taylor_six * a) * taylor_six.inverse = a
    real_mul_inverse_comm(taylor_six, b)
    (taylor_six * b) * taylor_six.inverse = b
    real_mul_inverse_comm(taylor_six, c)
    (taylor_six * c) * taylor_six.inverse = c
    // (3d) * 6.inverse = d * 2.inverse
    mul_assoc(taylor_three, d, taylor_six.inverse)
    (taylor_three * d) * taylor_six.inverse = taylor_three * (d * taylor_six.inverse)
    inverse_dist(taylor_two, taylor_three)
    (taylor_two * taylor_three).inverse = taylor_three.inverse * taylor_two.inverse
    taylor_six = taylor_two * taylor_three
    taylor_six.inverse = taylor_three.inverse * taylor_two.inverse
    d * taylor_six.inverse = d * (taylor_three.inverse * taylor_two.inverse)
    mul_assoc(d, taylor_three.inverse, taylor_two.inverse)
    d * (taylor_three.inverse * taylor_two.inverse) = (d * taylor_three.inverse) * taylor_two.inverse
    real_mul_comm(d, taylor_three.inverse)
    d * taylor_three.inverse = taylor_three.inverse * d
    (d * taylor_three.inverse) * taylor_two.inverse =
        (taylor_three.inverse * d) * taylor_two.inverse
    mul_assoc(taylor_three.inverse, d, taylor_two.inverse)
    (taylor_three.inverse * d) * taylor_two.inverse =
        taylor_three.inverse * (d * taylor_two.inverse)
    taylor_three * (d * taylor_six.inverse) =
        taylor_three * (taylor_three.inverse * (d * taylor_two.inverse))
    mul_assoc(taylor_three, taylor_three.inverse, d * taylor_two.inverse)
    taylor_three * (taylor_three.inverse * (d * taylor_two.inverse)) =
        (taylor_three * taylor_three.inverse) * (d * taylor_two.inverse)
    taylor_three_not_zero
    mul_inverse(taylor_three)
    taylor_three * taylor_three.inverse = Real.1
    (taylor_three * taylor_three.inverse) * (d * taylor_two.inverse) =
        Real.1 * (d * taylor_two.inverse)
    mul_one_left(d * taylor_two.inverse)
    Real.1 * (d * taylor_two.inverse) = d * taylor_two.inverse
    taylor_three * (d * taylor_six.inverse) = d * taylor_two.inverse
    (taylor_three * d) * taylor_six.inverse = d * taylor_two.inverse
    d * taylor_two.inverse = d / taylor_two
    (taylor_three * d) * taylor_six.inverse = d / taylor_two
    e * taylor_six.inverse = e / taylor_six
    (taylor_six * a - taylor_six * b - taylor_six * c - taylor_three * d - e) * taylor_six.inverse =
        a - b - c - d / taylor_two - e / taylor_six
    (taylor_six * a - taylor_six * b - taylor_six * c - taylor_three * d - e) / taylor_six =
        a - b - c - d / taylor_two - e / taylor_six
}

/// A product divided by four and then by six equals the product divided by
/// twenty-four.
theorem taylor4_div_24_prod(x: Real, y: Real) {
    ((x / taylor_four) * y) / taylor_six = (x / taylor_24) * y
} by {
    ((x / taylor_four) * y) / taylor_six =
        ((x / taylor_four) * y) * taylor_six.inverse
    x / taylor_four = x * taylor_four.inverse
    ((x / taylor_four) * y) = (x * taylor_four.inverse) * y
    ((x / taylor_four) * y) * taylor_six.inverse =
        ((x * taylor_four.inverse) * y) * taylor_six.inverse
    mul_assoc(x, taylor_four.inverse, y)
    (x * taylor_four.inverse) * y = x * (taylor_four.inverse * y)
    ((x * taylor_four.inverse) * y) * taylor_six.inverse =
        (x * (taylor_four.inverse * y)) * taylor_six.inverse
    mul_assoc(x, taylor_four.inverse * y, taylor_six.inverse)
    (x * (taylor_four.inverse * y)) * taylor_six.inverse =
        x * ((taylor_four.inverse * y) * taylor_six.inverse)
    mul_assoc(taylor_four.inverse, y, taylor_six.inverse)
    (taylor_four.inverse * y) * taylor_six.inverse =
        taylor_four.inverse * (y * taylor_six.inverse)
    real_mul_comm(y, taylor_six.inverse)
    y * taylor_six.inverse = taylor_six.inverse * y
    taylor_four.inverse * (y * taylor_six.inverse) =
        taylor_four.inverse * (taylor_six.inverse * y)
    mul_assoc(taylor_four.inverse, taylor_six.inverse, y)
    taylor_four.inverse * (taylor_six.inverse * y) =
        (taylor_four.inverse * taylor_six.inverse) * y
    (taylor_four.inverse * y) * taylor_six.inverse =
        (taylor_four.inverse * taylor_six.inverse) * y
    x * ((taylor_four.inverse * y) * taylor_six.inverse) =
        x * ((taylor_four.inverse * taylor_six.inverse) * y)
    mul_assoc(x, taylor_four.inverse * taylor_six.inverse, y)
    x * ((taylor_four.inverse * taylor_six.inverse) * y) =
        (x * (taylor_four.inverse * taylor_six.inverse)) * y
    inverse_dist(taylor_four, taylor_six)
    (taylor_four * taylor_six).inverse = taylor_six.inverse * taylor_four.inverse
    x * (taylor_four.inverse * taylor_six.inverse) = x * (taylor_four * taylor_six).inverse
    taylor_24 = taylor_four * taylor_six
    (taylor_four * taylor_six).inverse = taylor_24.inverse
    x * (taylor_four.inverse * taylor_six.inverse) = x * taylor_24.inverse
    (x * (taylor_four.inverse * taylor_six.inverse)) * y = (x * taylor_24.inverse) * y
    ((x / taylor_four) * y) * taylor_six.inverse = (x * taylor_24.inverse) * y
    x * taylor_24.inverse = x / taylor_24
    (x * taylor_24.inverse) * y = (x / taylor_24) * y
    ((x / taylor_four) * y) * taylor_six.inverse = (x / taylor_24) * y
    ((x / taylor_four) * y) / taylor_six = (x / taylor_24) * y
}

/// If the derivative of the auxiliary function of the fourth-order Taylor
/// theorem vanishes at c, the fourth-order Taylor formula holds at b.
theorem taylor4_conclusion_from_derivative(f: Real -> Real, df: Real -> Real, ddf: Real -> Real, dddf: Real -> Real, dddd: Real -> Real, a: Real, b: Real, c: Real) {
    a < c and c < b and taylor4_aux_derivative(f, df, ddf, dddf, dddd, a, b, c) = Real.0
    implies f(b) = f(a) + df(a) * (b - a) + ddf(a) * (b - a).pow(Nat.2) / taylor_two +
        dddf(a) * (b - a).pow(Nat.3) / taylor_six + (dddd(c) / taylor_24) * (b - a).pow(Nat.4)
} by {
    if a < c and c < b and taylor4_aux_derivative(f, df, ddf, dddf, dddd, a, b, c) = Real.0 {
        taylor4_aux_derivative(f, df, ddf, dddf, dddd, a, b, c) =
            (b - c).pow(Nat.3) * (taylor_four * taylor4_remainder_coeff(f, df, ddf, dddf, a, b) - dddd(c))
        (b - c).pow(Nat.3) * (taylor_four * taylor4_remainder_coeff(f, df, ddf, dddf, a, b) - dddd(c)) = Real.0
        lt_imp_ne_symm(c, b)
        b != c
        sub_ne_zero_of_ne(b, c)
        b - c != Real.0
        field_square_nonzero[Real](b - c)
        (b - c) * (b - c) != Real.0
        mul_not_zero[Real](b - c, (b - c) * (b - c))
        (b - c) * ((b - c) * (b - c)) != Real.0
        real_pow_three_eq_mul(b - c)
        (b - c).pow(Nat.3) = (b - c) * (b - c) * (b - c)
        (b - c).pow(Nat.3) != Real.0
        mul_zero_right(taylor_four * taylor4_remainder_coeff(f, df, ddf, dddf, a, b) - dddd(c))
        (taylor_four * taylor4_remainder_coeff(f, df, ddf, dddf, a, b) - dddd(c)) * Real.0 = Real.0
        mul_left_cancel((b - c).pow(Nat.3),
            taylor_four * taylor4_remainder_coeff(f, df, ddf, dddf, a, b) - dddd(c), Real.0)
        taylor_four * taylor4_remainder_coeff(f, df, ddf, dddf, a, b) - dddd(c) = Real.0
        sub_zero_imp_eq(taylor_four * taylor4_remainder_coeff(f, df, ddf, dddf, a, b), dddd(c))
        taylor_four * taylor4_remainder_coeff(f, df, ddf, dddf, a, b) = dddd(c)
        taylor_four_not_zero
        taylor_four != Real.0
        mul_inverse(taylor_four)
        taylor_four * taylor_four.inverse = Real.1
        (taylor_four * taylor4_remainder_coeff(f, df, ddf, dddf, a, b)) * taylor_four.inverse =
            dddd(c) * taylor_four.inverse
        mul_assoc(taylor_four, taylor4_remainder_coeff(f, df, ddf, dddf, a, b), taylor_four.inverse)
        (taylor_four * taylor4_remainder_coeff(f, df, ddf, dddf, a, b)) * taylor_four.inverse =
            taylor_four * (taylor4_remainder_coeff(f, df, ddf, dddf, a, b) * taylor_four.inverse)
        real_mul_comm(taylor4_remainder_coeff(f, df, ddf, dddf, a, b), taylor_four.inverse)
        taylor4_remainder_coeff(f, df, ddf, dddf, a, b) * taylor_four.inverse =
            taylor_four.inverse * taylor4_remainder_coeff(f, df, ddf, dddf, a, b)
        mul_assoc(taylor_four, taylor_four.inverse, taylor4_remainder_coeff(f, df, ddf, dddf, a, b))
        taylor_four * (taylor_four.inverse * taylor4_remainder_coeff(f, df, ddf, dddf, a, b)) =
            (taylor_four * taylor_four.inverse) * taylor4_remainder_coeff(f, df, ddf, dddf, a, b)
        (taylor_four * taylor_four.inverse) * taylor4_remainder_coeff(f, df, ddf, dddf, a, b) =
            Real.1 * taylor4_remainder_coeff(f, df, ddf, dddf, a, b)
        mul_one_left(taylor4_remainder_coeff(f, df, ddf, dddf, a, b))
        Real.1 * taylor4_remainder_coeff(f, df, ddf, dddf, a, b) =
            taylor4_remainder_coeff(f, df, ddf, dddf, a, b)
        (taylor_four * taylor4_remainder_coeff(f, df, ddf, dddf, a, b)) * taylor_four.inverse =
            taylor4_remainder_coeff(f, df, ddf, dddf, a, b)
        taylor4_remainder_coeff(f, df, ddf, dddf, a, b) = dddd(c) * taylor_four.inverse
        taylor4_remainder_coeff(f, df, ddf, dddf, a, b) = dddd(c) / taylor_four
        lt_trans[Real](a, c, b)
        a < b
        lt_imp_ne_symm(a, b)
        b != a
        sub_ne_zero_of_ne(b, a)
        b - a != Real.0
        field_square_nonzero[Real](b - a)
        (b - a) * (b - a) != Real.0
        mul_not_zero[Real]((b - a) * (b - a), (b - a) * (b - a))
        ((b - a) * (b - a)) * ((b - a) * (b - a)) != Real.0
        real_pow_four_eq_mul(b - a)
        (b - a).pow(Nat.4) = ((b - a) * (b - a)) * ((b - a) * (b - a))
        (b - a).pow(Nat.4) != Real.0
        taylor4_remainder_coeff(f, df, ddf, dddf, a, b) =
            taylor4_doubled_delta(f, df, ddf, dddf, a, b) / (b - a).pow(Nat.4)
        div_mul_cancel_denominator(taylor4_doubled_delta(f, df, ddf, dddf, a, b), (b - a).pow(Nat.4))
        (taylor4_doubled_delta(f, df, ddf, dddf, a, b) / (b - a).pow(Nat.4)) * (b - a).pow(Nat.4) =
            taylor4_doubled_delta(f, df, ddf, dddf, a, b)
        taylor4_remainder_coeff(f, df, ddf, dddf, a, b) * (b - a).pow(Nat.4) =
            taylor4_doubled_delta(f, df, ddf, dddf, a, b)
        taylor4_doubled_delta(f, df, ddf, dddf, a, b) =
            taylor_six * f(b) - taylor_six * f(a) - taylor_six * df(a) * (b - a) -
                taylor_three * ddf(a) * (b - a).pow(Nat.2) - dddf(a) * (b - a).pow(Nat.3)
        taylor_six * f(b) - taylor_six * f(a) - taylor_six * df(a) * (b - a) -
            taylor_three * ddf(a) * (b - a).pow(Nat.2) - dddf(a) * (b - a).pow(Nat.3) =
            (dddd(c) / taylor_four) * (b - a).pow(Nat.4)
        taylor_six_not_zero
        taylor_six != Real.0
        (taylor_six * f(b) - taylor_six * f(a) - taylor_six * df(a) * (b - a) -
            taylor_three * ddf(a) * (b - a).pow(Nat.2) - dddf(a) * (b - a).pow(Nat.3)) / taylor_six =
            ((dddd(c) / taylor_four) * (b - a).pow(Nat.4)) / taylor_six
        taylor4_div_six_chain(f(b), f(a), df(a) * (b - a), ddf(a) * (b - a).pow(Nat.2),
            dddf(a) * (b - a).pow(Nat.3))
        (taylor_six * f(b) - taylor_six * f(a) - taylor_six * (df(a) * (b - a)) -
            taylor_three * (ddf(a) * (b - a).pow(Nat.2)) - dddf(a) * (b - a).pow(Nat.3)) / taylor_six =
            f(b) - f(a) - df(a) * (b - a) - ddf(a) * (b - a).pow(Nat.2) / taylor_two -
                dddf(a) * (b - a).pow(Nat.3) / taylor_six
        mul_assoc(taylor_six, df(a), b - a)
        taylor_six * df(a) * (b - a) = taylor_six * (df(a) * (b - a))
        mul_assoc(taylor_three, ddf(a), (b - a).pow(Nat.2))
        taylor_three * ddf(a) * (b - a).pow(Nat.2) =
            taylor_three * (ddf(a) * (b - a).pow(Nat.2))
        (taylor_six * f(b) - taylor_six * f(a) - taylor_six * df(a) * (b - a) -
            taylor_three * ddf(a) * (b - a).pow(Nat.2) - dddf(a) * (b - a).pow(Nat.3)) / taylor_six =
            (taylor_six * f(b) - taylor_six * f(a) - taylor_six * (df(a) * (b - a)) -
                taylor_three * (ddf(a) * (b - a).pow(Nat.2)) - dddf(a) * (b - a).pow(Nat.3)) / taylor_six
        (taylor_six * f(b) - taylor_six * f(a) - taylor_six * df(a) * (b - a) -
            taylor_three * ddf(a) * (b - a).pow(Nat.2) - dddf(a) * (b - a).pow(Nat.3)) / taylor_six =
            f(b) - f(a) - df(a) * (b - a) - ddf(a) * (b - a).pow(Nat.2) / taylor_two -
                dddf(a) * (b - a).pow(Nat.3) / taylor_six
        taylor4_div_24_prod(dddd(c), (b - a).pow(Nat.4))
        ((dddd(c) / taylor_four) * (b - a).pow(Nat.4)) / taylor_six =
            (dddd(c) / taylor_24) * (b - a).pow(Nat.4)
        f(b) - f(a) - df(a) * (b - a) - ddf(a) * (b - a).pow(Nat.2) / taylor_two -
            dddf(a) * (b - a).pow(Nat.3) / taylor_six =
            (dddd(c) / taylor_24) * (b - a).pow(Nat.4)
        taylor4_sub_add_cancel4(f(b), f(a), df(a) * (b - a),
            ddf(a) * (b - a).pow(Nat.2) / taylor_two, dddf(a) * (b - a).pow(Nat.3) / taylor_six)
        (f(b) - f(a) - df(a) * (b - a) - ddf(a) * (b - a).pow(Nat.2) / taylor_two -
            dddf(a) * (b - a).pow(Nat.3) / taylor_six) + f(a) + df(a) * (b - a) +
            ddf(a) * (b - a).pow(Nat.2) / taylor_two + dddf(a) * (b - a).pow(Nat.3) / taylor_six = f(b)
        (f(b) - f(a) - df(a) * (b - a) - ddf(a) * (b - a).pow(Nat.2) / taylor_two -
            dddf(a) * (b - a).pow(Nat.3) / taylor_six) +
            (f(a) + df(a) * (b - a) + ddf(a) * (b - a).pow(Nat.2) / taylor_two +
                dddf(a) * (b - a).pow(Nat.3) / taylor_six) =
            (dddd(c) / taylor_24) * (b - a).pow(Nat.4) +
                (f(a) + df(a) * (b - a) + ddf(a) * (b - a).pow(Nat.2) / taylor_two +
                    dddf(a) * (b - a).pow(Nat.3) / taylor_six)
        add_assoc(f(b) - f(a) - df(a) * (b - a) - ddf(a) * (b - a).pow(Nat.2) / taylor_two -
                dddf(a) * (b - a).pow(Nat.3) / taylor_six,
            f(a), df(a) * (b - a) + ddf(a) * (b - a).pow(Nat.2) / taylor_two +
                dddf(a) * (b - a).pow(Nat.3) / taylor_six)
        (f(b) - f(a) - df(a) * (b - a) - ddf(a) * (b - a).pow(Nat.2) / taylor_two -
            dddf(a) * (b - a).pow(Nat.3) / taylor_six) +
            (f(a) + df(a) * (b - a) + ddf(a) * (b - a).pow(Nat.2) / taylor_two +
                dddf(a) * (b - a).pow(Nat.3) / taylor_six) = f(b)
        f(b) = (dddd(c) / taylor_24) * (b - a).pow(Nat.4) +
            (f(a) + df(a) * (b - a) + ddf(a) * (b - a).pow(Nat.2) / taylor_two +
                dddf(a) * (b - a).pow(Nat.3) / taylor_six)
        add_comm((dddd(c) / taylor_24) * (b - a).pow(Nat.4),
            f(a) + df(a) * (b - a) + ddf(a) * (b - a).pow(Nat.2) / taylor_two +
                dddf(a) * (b - a).pow(Nat.3) / taylor_six)
        (dddd(c) / taylor_24) * (b - a).pow(Nat.4) +
            (f(a) + df(a) * (b - a) + ddf(a) * (b - a).pow(Nat.2) / taylor_two +
                dddf(a) * (b - a).pow(Nat.3) / taylor_six) =
            f(a) + df(a) * (b - a) + ddf(a) * (b - a).pow(Nat.2) / taylor_two +
                dddf(a) * (b - a).pow(Nat.3) / taylor_six + (dddd(c) / taylor_24) * (b - a).pow(Nat.4)
        f(b) = f(a) + df(a) * (b - a) + ddf(a) * (b - a).pow(Nat.2) / taylor_two +
            dddf(a) * (b - a).pow(Nat.3) / taylor_six + (dddd(c) / taylor_24) * (b - a).pow(Nat.4)
    }
}

/// Taylor's theorem of order three with Lagrange remainder, in the
/// iterated-derivative notation: if f has four derivatives everywhere, then
/// the value at b equals the third-order Taylor polynomial at a plus the
/// remainder f''''(c) (b - a)^4 / 24 for some interior c.
theorem taylor_general_three(f: Real -> Real, df: Real -> Real, ddf: Real -> Real, dddf: Real -> Real, dddd: Real -> Real, a: Real, b: Real) {
    a < b and is_derivative_fn(f, df) and is_derivative_fn(df, ddf) and is_derivative_fn(ddf, dddf) and is_derivative_fn(dddf, dddd)
    implies exists(c: Real) {
        a < c and c < b and
        f(b) = taylor_poly(f, taylor_chain4(f, df, ddf, dddf, dddd), a, b, Nat.3) +
            taylor_remainder(f, taylor_chain4(f, df, ddf, dddf, dddd), a, b, c, Nat.3)
    }
} by {
    if a < b and is_derivative_fn(f, df) and is_derivative_fn(df, ddf) and is_derivative_fn(ddf, dddf) and is_derivative_fn(dddf, dddd) {
        taylor4_aux_continuous_on_closed(f, df, ddf, dddf, dddd, a, b, a, b)
        continuous_on_closed(taylor4_aux(f, df, ddf, dddf, a, b), a, b)
        taylor4_aux_is_derivative_fn(f, df, ddf, dddf, dddd, a, b)
        is_derivative_fn(taylor4_aux(f, df, ddf, dddf, a, b),
            taylor4_aux_derivative(f, df, ddf, dddf, dddd, a, b))
        is_derivative_fn_imp_is_derivative_on_open(taylor4_aux(f, df, ddf, dddf, a, b),
            taylor4_aux_derivative(f, df, ddf, dddf, dddd, a, b), a, b)
        is_derivative_on_open(taylor4_aux(f, df, ddf, dddf, a, b),
            taylor4_aux_derivative(f, df, ddf, dddf, dddd, a, b), a, b)
        is_derivative_on_open_imp_differentiable_on_open(taylor4_aux(f, df, ddf, dddf, a, b),
            taylor4_aux_derivative(f, df, ddf, dddf, dddd, a, b), a, b)
        differentiable_on_open(taylor4_aux(f, df, ddf, dddf, a, b), a, b)
        taylor4_aux_endpoints_equal(f, df, ddf, dddf, a, b)
        taylor4_aux(f, df, ddf, dddf, a, b, a) = taylor4_aux(f, df, ddf, dddf, a, b, b)
        rolle_theorem(taylor4_aux(f, df, ddf, dddf, a, b), a, b)
        let c1: Real satisfy {
            a < c1 and c1 < b and
            has_derivative_at(taylor4_aux(f, df, ddf, dddf, a, b), c1, Real.0)
        }
        a < c1 and c1 < b
        has_derivative_at(taylor4_aux(f, df, ddf, dddf, a, b), c1, Real.0)
        is_derivative_fn_at(taylor4_aux(f, df, ddf, dddf, a, b),
            taylor4_aux_derivative(f, df, ddf, dddf, dddd, a, b), c1)
        has_derivative_at(taylor4_aux(f, df, ddf, dddf, a, b), c1,
            taylor4_aux_derivative(f, df, ddf, dddf, dddd, a, b, c1))
        has_derivative_at_unique(taylor4_aux(f, df, ddf, dddf, a, b), c1, Real.0,
            taylor4_aux_derivative(f, df, ddf, dddf, dddd, a, b, c1))
        Real.0 = taylor4_aux_derivative(f, df, ddf, dddf, dddd, a, b, c1)
        taylor4_aux_derivative(f, df, ddf, dddf, dddd, a, b, c1) = Real.0
        taylor4_conclusion_from_derivative(f, df, ddf, dddf, dddd, a, b, c1)
        f(b) = f(a) + df(a) * (b - a) + ddf(a) * (b - a).pow(Nat.2) / taylor_two +
            dddf(a) * (b - a).pow(Nat.3) / taylor_six + (dddd(c1) / taylor_24) * (b - a).pow(Nat.4)
        taylor_poly(f, taylor_chain4(f, df, ddf, dddf, dddd), a, b, Nat.3) =
            partial(taylor_term(f, taylor_chain4(f, df, ddf, dddf, dddd), a, b), Nat.4)
        partial_four_sum(taylor_term(f, taylor_chain4(f, df, ddf, dddf, dddd), a, b))
        partial(taylor_term(f, taylor_chain4(f, df, ddf, dddf, dddd), a, b), Nat.4) =
            taylor_term(f, taylor_chain4(f, df, ddf, dddf, dddd), a, b, Nat.0) +
            taylor_term(f, taylor_chain4(f, df, ddf, dddf, dddd), a, b, Nat.1) +
            taylor_term(f, taylor_chain4(f, df, ddf, dddf, dddd), a, b, Nat.2) +
            taylor_term(f, taylor_chain4(f, df, ddf, dddf, dddd), a, b, Nat.3)
        taylor_poly(f, taylor_chain4(f, df, ddf, dddf, dddd), a, b, Nat.3) =
            taylor_term(f, taylor_chain4(f, df, ddf, dddf, dddd), a, b, Nat.0) +
            taylor_term(f, taylor_chain4(f, df, ddf, dddf, dddd), a, b, Nat.1) +
            taylor_term(f, taylor_chain4(f, df, ddf, dddf, dddd), a, b, Nat.2) +
            taylor_term(f, taylor_chain4(f, df, ddf, dddf, dddd), a, b, Nat.3)
        taylor_term(f, taylor_chain4(f, df, ddf, dddf, dddd), a, b, Nat.0) =
            (iterated_derivative(f, taylor_chain4(f, df, ddf, dddf, dddd), Nat.0, a) /
                from_nat[Real](Nat.0.factorial)) * (b - a).pow(Nat.0)
        iterated_derivative(f, taylor_chain4(f, df, ddf, dddf, dddd), Nat.0, a) =
            taylor_chain4(f, df, ddf, dddf, dddd, Nat.0, a)
        taylor_chain4_zero(f, df, ddf, dddf, dddd)
        taylor_chain4(f, df, ddf, dddf, dddd, Nat.0) = f
        taylor_chain4(f, df, ddf, dddf, dddd, Nat.0, a) = f(a)
        iterated_derivative(f, taylor_chain4(f, df, ddf, dddf, dddd), Nat.0, a) = f(a)
        from_nat_factorial_zero_real
        from_nat[Real](Nat.0.factorial) = Real.1
        real_pow_zero_eq(b - a)
        (b - a).pow(Nat.0) = Real.1
        taylor_term(f, taylor_chain4(f, df, ddf, dddf, dddd), a, b, Nat.0) = (f(a) / Real.1) * Real.1
        real_div_one(f(a))
        f(a) / Real.1 = f(a)
        (f(a) / Real.1) * Real.1 = f(a) * Real.1
        mul_one_right(f(a))
        f(a) * Real.1 = f(a)
        (f(a) / Real.1) * Real.1 = f(a)
        taylor_term(f, taylor_chain4(f, df, ddf, dddf, dddd), a, b, Nat.0) = f(a)
        taylor_term(f, taylor_chain4(f, df, ddf, dddf, dddd), a, b, Nat.1) =
            (iterated_derivative(f, taylor_chain4(f, df, ddf, dddf, dddd), Nat.1, a) /
                from_nat[Real](Nat.1.factorial)) * (b - a).pow(Nat.1)
        iterated_derivative(f, taylor_chain4(f, df, ddf, dddf, dddd), Nat.1, a) =
            taylor_chain4(f, df, ddf, dddf, dddd, Nat.1, a)
        taylor_chain4_one(f, df, ddf, dddf, dddd)
        taylor_chain4(f, df, ddf, dddf, dddd, Nat.1) = df
        taylor_chain4(f, df, ddf, dddf, dddd, Nat.1, a) = df(a)
        iterated_derivative(f, taylor_chain4(f, df, ddf, dddf, dddd), Nat.1, a) = df(a)
        from_nat_factorial_one_real
        from_nat[Real](Nat.1.factorial) = Real.1
        real_pow_one_eq(b - a)
        (b - a).pow(Nat.1) = b - a
        taylor_term(f, taylor_chain4(f, df, ddf, dddf, dddd), a, b, Nat.1) =
            (df(a) / Real.1) * (b - a)
        real_div_one(df(a))
        df(a) / Real.1 = df(a)
        (df(a) / Real.1) * (b - a) = df(a) * (b - a)
        taylor_term(f, taylor_chain4(f, df, ddf, dddf, dddd), a, b, Nat.1) = df(a) * (b - a)
        taylor_term(f, taylor_chain4(f, df, ddf, dddf, dddd), a, b, Nat.2) =
            (iterated_derivative(f, taylor_chain4(f, df, ddf, dddf, dddd), Nat.2, a) /
                from_nat[Real](Nat.2.factorial)) * (b - a).pow(Nat.2)
        iterated_derivative(f, taylor_chain4(f, df, ddf, dddf, dddd), Nat.2, a) =
            taylor_chain4(f, df, ddf, dddf, dddd, Nat.2, a)
        taylor_chain4_two(f, df, ddf, dddf, dddd)
        taylor_chain4(f, df, ddf, dddf, dddd, Nat.2) = ddf
        taylor_chain4(f, df, ddf, dddf, dddd, Nat.2, a) = ddf(a)
        iterated_derivative(f, taylor_chain4(f, df, ddf, dddf, dddd), Nat.2, a) = ddf(a)
        from_nat_factorial_two_real
        from_nat[Real](Nat.2.factorial) = taylor_two
        taylor_term(f, taylor_chain4(f, df, ddf, dddf, dddd), a, b, Nat.2) =
            (ddf(a) / taylor_two) * (b - a).pow(Nat.2)
        taylor_term(f, taylor_chain4(f, df, ddf, dddf, dddd), a, b, Nat.3) =
            (iterated_derivative(f, taylor_chain4(f, df, ddf, dddf, dddd), Nat.3, a) /
                from_nat[Real](Nat.3.factorial)) * (b - a).pow(Nat.3)
        iterated_derivative(f, taylor_chain4(f, df, ddf, dddf, dddd), Nat.3, a) =
            taylor_chain4(f, df, ddf, dddf, dddd, Nat.3, a)
        taylor_chain4_three(f, df, ddf, dddf, dddd)
        taylor_chain4(f, df, ddf, dddf, dddd, Nat.3) = dddf
        taylor_chain4(f, df, ddf, dddf, dddd, Nat.3, a) = dddf(a)
        iterated_derivative(f, taylor_chain4(f, df, ddf, dddf, dddd), Nat.3, a) = dddf(a)
        from_nat_factorial_three_real
        from_nat[Real](Nat.3.factorial) = taylor_six
        taylor_term(f, taylor_chain4(f, df, ddf, dddf, dddd), a, b, Nat.3) =
            (dddf(a) / taylor_six) * (b - a).pow(Nat.3)
        taylor_poly(f, taylor_chain4(f, df, ddf, dddf, dddd), a, b, Nat.3) =
            f(a) + df(a) * (b - a) + (ddf(a) / taylor_two) * (b - a).pow(Nat.2) +
                (dddf(a) / taylor_six) * (b - a).pow(Nat.3)
        taylor_remainder(f, taylor_chain4(f, df, ddf, dddf, dddd), a, b, c1, Nat.3) =
            (iterated_derivative(f, taylor_chain4(f, df, ddf, dddf, dddd), Nat.3.suc, c1) /
                from_nat[Real](Nat.3.suc.factorial)) * (b - a).pow(Nat.3.suc)
        Nat.3.suc = Nat.4
        taylor_remainder(f, taylor_chain4(f, df, ddf, dddf, dddd), a, b, c1, Nat.3) =
            (iterated_derivative(f, taylor_chain4(f, df, ddf, dddf, dddd), Nat.4, c1) /
                from_nat[Real](Nat.4.factorial)) * (b - a).pow(Nat.4)
        iterated_derivative(f, taylor_chain4(f, df, ddf, dddf, dddd), Nat.4, c1) =
            taylor_chain4(f, df, ddf, dddf, dddd, Nat.4, c1)
        taylor_chain4_four(f, df, ddf, dddf, dddd)
        taylor_chain4(f, df, ddf, dddf, dddd, Nat.4) = dddd
        taylor_chain4(f, df, ddf, dddf, dddd, Nat.4, c1) = dddd(c1)
        iterated_derivative(f, taylor_chain4(f, df, ddf, dddf, dddd), Nat.4, c1) = dddd(c1)
        from_nat_factorial_four_real
        from_nat[Real](Nat.4.factorial) = taylor_24
        taylor_remainder(f, taylor_chain4(f, df, ddf, dddf, dddd), a, b, c1, Nat.3) =
            (dddd(c1) / taylor_24) * (b - a).pow(Nat.4)
        ddf(a) / taylor_two = ddf(a) * taylor_two.inverse
        (ddf(a) / taylor_two) * (b - a).pow(Nat.2) =
            (ddf(a) * taylor_two.inverse) * (b - a).pow(Nat.2)
        mul_assoc(ddf(a), taylor_two.inverse, (b - a).pow(Nat.2))
        (ddf(a) * taylor_two.inverse) * (b - a).pow(Nat.2) =
            ddf(a) * (taylor_two.inverse * (b - a).pow(Nat.2))
        real_mul_comm(taylor_two.inverse, (b - a).pow(Nat.2))
        taylor_two.inverse * (b - a).pow(Nat.2) =
            (b - a).pow(Nat.2) * taylor_two.inverse
        ddf(a) * (taylor_two.inverse * (b - a).pow(Nat.2)) =
            ddf(a) * ((b - a).pow(Nat.2) * taylor_two.inverse)
        mul_assoc(ddf(a), (b - a).pow(Nat.2), taylor_two.inverse)
        ddf(a) * ((b - a).pow(Nat.2) * taylor_two.inverse) =
            (ddf(a) * (b - a).pow(Nat.2)) * taylor_two.inverse
        (ddf(a) / taylor_two) * (b - a).pow(Nat.2) =
            (ddf(a) * (b - a).pow(Nat.2)) * taylor_two.inverse
        ddf(a) * (b - a).pow(Nat.2) / taylor_two =
            (ddf(a) * (b - a).pow(Nat.2)) * taylor_two.inverse
        (ddf(a) / taylor_two) * (b - a).pow(Nat.2) =
            ddf(a) * (b - a).pow(Nat.2) / taylor_two
        dddf(a) / taylor_six = dddf(a) * taylor_six.inverse
        (dddf(a) / taylor_six) * (b - a).pow(Nat.3) =
            (dddf(a) * taylor_six.inverse) * (b - a).pow(Nat.3)
        mul_assoc(dddf(a), taylor_six.inverse, (b - a).pow(Nat.3))
        (dddf(a) * taylor_six.inverse) * (b - a).pow(Nat.3) =
            dddf(a) * (taylor_six.inverse * (b - a).pow(Nat.3))
        real_mul_comm(taylor_six.inverse, (b - a).pow(Nat.3))
        taylor_six.inverse * (b - a).pow(Nat.3) =
            (b - a).pow(Nat.3) * taylor_six.inverse
        dddf(a) * (taylor_six.inverse * (b - a).pow(Nat.3)) =
            dddf(a) * ((b - a).pow(Nat.3) * taylor_six.inverse)
        mul_assoc(dddf(a), (b - a).pow(Nat.3), taylor_six.inverse)
        dddf(a) * ((b - a).pow(Nat.3) * taylor_six.inverse) =
            (dddf(a) * (b - a).pow(Nat.3)) * taylor_six.inverse
        (dddf(a) / taylor_six) * (b - a).pow(Nat.3) =
            (dddf(a) * (b - a).pow(Nat.3)) * taylor_six.inverse
        dddf(a) * (b - a).pow(Nat.3) / taylor_six =
            (dddf(a) * (b - a).pow(Nat.3)) * taylor_six.inverse
        (dddf(a) / taylor_six) * (b - a).pow(Nat.3) =
            dddf(a) * (b - a).pow(Nat.3) / taylor_six
        f(a) + df(a) * (b - a) + (ddf(a) / taylor_two) * (b - a).pow(Nat.2) +
            (dddf(a) / taylor_six) * (b - a).pow(Nat.3) +
            (dddd(c1) / taylor_24) * (b - a).pow(Nat.4) =
            f(a) + df(a) * (b - a) + ddf(a) * (b - a).pow(Nat.2) / taylor_two +
                dddf(a) * (b - a).pow(Nat.3) / taylor_six +
                (dddd(c1) / taylor_24) * (b - a).pow(Nat.4)
        taylor_poly(f, taylor_chain4(f, df, ddf, dddf, dddd), a, b, Nat.3) +
            taylor_remainder(f, taylor_chain4(f, df, ddf, dddf, dddd), a, b, c1, Nat.3) =
            f(a) + df(a) * (b - a) + ddf(a) * (b - a).pow(Nat.2) / taylor_two +
                dddf(a) * (b - a).pow(Nat.3) / taylor_six +
                (dddd(c1) / taylor_24) * (b - a).pow(Nat.4)
        f(b) = taylor_poly(f, taylor_chain4(f, df, ddf, dddf, dddd), a, b, Nat.3) +
            taylor_remainder(f, taylor_chain4(f, df, ddf, dddf, dddd), a, b, c1, Nat.3)
        exists(c2: Real) {
            a < c2 and c2 < b and
            f(b) = taylor_poly(f, taylor_chain4(f, df, ddf, dddf, dddd), a, b, Nat.3) +
                taylor_remainder(f, taylor_chain4(f, df, ddf, dddf, dddd), a, b, c2, Nat.3)
        }
    }
}

// General statement: if f has n + 1 derivatives on [a, b] (encoded by a chain
// dfs of derivative functions), then the value at b equals the n-th order
// Taylor polynomial at a plus the Lagrange remainder of order n + 1.
//
// theorem taylor_general(n: Nat, f: Real -> Real, dfs: Nat -> Real -> Real, a: Real, b: Real) {
//     a < b and is_derivative_chain(f, dfs, n + 1)
//     implies exists(c: Real) {
//         a < c and c < b and
//         f(b) = taylor_poly(f, dfs, a, b, n) + taylor_remainder(f, dfs, a, b, c, n)
//     }
// }
//
// The classical proof applies Rolle's theorem to the auxiliary function
// g(x) = f(b) - sum_{k=0}^{n} f^(k)(x) (b - x)^k / k! - M (b - x)^(n + 1) with
// M chosen so that g(a) = 0.  Its derivative telescopes to
// (b - x)^n ((n + 1) M - f^(n+1)(x) / n!), which needs a telescoping identity
// for partial sums over the variable range k = 0..n.  That infrastructure is
// not yet present; the instances at n = 0, 1, 2 and 3 are proved directly in
// this file.

