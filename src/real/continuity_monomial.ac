from data.basic.functions import function_extensionality
from nat import Nat
from real.continuity_const_mul import const_mul_left, continuous_const_mul_left
from real.continuity_pow import continuous_pow_real_fn, pow_real_fn
from real.continuity_base import Real, continuous

/// The monomial function with coefficient c and degree n, mapping x to c * x^n.
define monomial_real_fn(c: Real, n: Nat, x: Real) -> Real {
    c * x.pow(n)
}

/// A monomial agrees with the coefficient multiplied on the left of the power function.
theorem monomial_real_fn_eq_const_mul_pow(c: Real, n: Nat) {
    monomial_real_fn(c, n) = const_mul_left(c, pow_real_fn(n))
} by {
    forall(x: Real) {
        monomial_real_fn(c, n, x) = c * x.pow(n)
        pow_real_fn(n, x) = x.pow(n)
        const_mul_left(c, pow_real_fn(n), x) = c * pow_real_fn(n, x)
        const_mul_left(c, pow_real_fn(n), x) = c * x.pow(n)
        monomial_real_fn(c, n, x) = const_mul_left(c, pow_real_fn(n), x)
    }
    function_extensionality(monomial_real_fn(c, n), const_mul_left(c, pow_real_fn(n)))
}

/// Every monomial function on the reals is continuous.
theorem continuous_monomial_real_fn(c: Real, n: Nat) {
    continuous(monomial_real_fn(c, n))
} by {
    continuous_pow_real_fn(n)
    continuous(pow_real_fn(n))
    continuous_const_mul_left(c, pow_real_fn(n))
    continuous(const_mul_left(c, pow_real_fn(n)))
    monomial_real_fn_eq_const_mul_pow(c, n)
    continuous(monomial_real_fn(c, n))
}
