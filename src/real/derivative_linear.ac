from data.basic.function_algebra import pointwise_add, pointwise_mul, pointwise_mul_comm
from data.basic.functions import identity_fn
from real.derivative_basic import difference_quotient, has_derivative_at,
    differentiable_at, constant_has_derivative_at, identity_has_derivative_at
from real.derivative_rules import derivative_pointwise_add
from real.real_base import Real, abs_not_neg, lt_trans
from real.real_ring import exists_small_mul_variant, lt_mul_pos_left, mul_abs

/// Multiplication by a fixed real preserves closeness after rescaling the tolerance.
theorem scalar_mul_close(c: Real, a: Real, b: Real, eps2: Real, eps: Real) {
    eps.is_positive and a.is_close(b, eps2) and c.abs * eps2 < eps
    implies (c * a).is_close(c * b, eps)
} by {
    a.is_close(b, eps2) = (a - b).abs < eps2
    (a - b).abs < eps2
    c * a - c * b = c * (a - b)
    mul_abs(c, a - b)
    (c * (a - b)).abs = c.abs * (a - b).abs
    (c * a - c * b).abs = c.abs * (a - b).abs
    if c.abs.is_positive {
        lt_mul_pos_left((a - b).abs, eps2, c.abs)
        c.abs * (a - b).abs < c.abs * eps2
        lt_trans(c.abs * (a - b).abs, c.abs * eps2, eps)
        c.abs * (a - b).abs < eps
    } else {
        abs_not_neg(c)
        not c.abs.is_negative
        c.abs = Real.0
        c.abs * (a - b).abs = Real.0
        Real.0 < eps
        c.abs * (a - b).abs < eps
    }
    (c * a - c * b).abs < eps
    (c * a).is_close(c * b, eps)
}

/// A scalar factor in the numerator may be pulled outside a quotient.
theorem mul_div_left(c: Real, a: Real, b: Real) {
    (c * a) / b = c * (a / b)
} by {
    (c * a) / b = (c * a) * b.inverse
    a / b = a * b.inverse
    c * (a / b) = c * (a * b.inverse)
    (c * a) * b.inverse = c * (a * b.inverse)
}

/// The difference quotient of a left scalar multiple is the scalar multiple of the difference quotient.
theorem difference_quotient_pointwise_const_mul(c: Real, f: Real -> Real, x0: Real, x: Real) {
    x != x0 implies difference_quotient(pointwise_mul(constant[Real, Real](c), f), x0, x) =
        c * difference_quotient(f, x0, x)
} by {
    if x != x0 {
        constant[Real, Real](c, x) = c
        constant[Real, Real](c, x0) = c
        pointwise_mul(constant[Real, Real](c), f, x) = c * f(x)
        pointwise_mul(constant[Real, Real](c), f, x0) = c * f(x0)
        let a = f(x) - f(x0)
        c * f(x) - c * f(x0) = c * a
        difference_quotient(pointwise_mul(constant[Real, Real](c), f), x0, x) = (c * a) / (x - x0)
        mul_div_left(c, a, x - x0)
        (c * a) / (x - x0) = c * (a / (x - x0))
        difference_quotient(f, x0, x) = a / (x - x0)
        difference_quotient(pointwise_mul(constant[Real, Real](c), f), x0, x) =
            c * difference_quotient(f, x0, x)
    }
}

/// The difference quotient of a right scalar multiple is the scalar multiple of the difference quotient.
theorem difference_quotient_pointwise_mul_const(c: Real, f: Real -> Real, x0: Real, x: Real) {
    x != x0 implies difference_quotient(pointwise_mul(f, constant[Real, Real](c)), x0, x) =
        difference_quotient(f, x0, x) * c
} by {
    if x != x0 {
        constant[Real, Real](c, x) = c
        constant[Real, Real](c, x0) = c
        pointwise_mul(f, constant[Real, Real](c), x) = f(x) * c
        pointwise_mul(f, constant[Real, Real](c), x0) = f(x0) * c
        let a = f(x) - f(x0)
        f(x) * c - f(x0) * c = a * c
        a * c = c * a
        difference_quotient(pointwise_mul(f, constant[Real, Real](c)), x0, x) = (c * a) / (x - x0)
        mul_div_left(c, a, x - x0)
        (c * a) / (x - x0) = c * (a / (x - x0))
        difference_quotient(f, x0, x) = a / (x - x0)
        c * difference_quotient(f, x0, x) = difference_quotient(f, x0, x) * c
        difference_quotient(pointwise_mul(f, constant[Real, Real](c)), x0, x) =
            difference_quotient(f, x0, x) * c
    }
}

/// Left scalar multiplication multiplies derivatives by that scalar.
theorem derivative_pointwise_const_mul(c: Real, f: Real -> Real, x0: Real, d: Real) {
    has_derivative_at(f, x0, d)
    implies has_derivative_at(pointwise_mul(constant[Real, Real](c), f), x0, c * d)
} by {
    has_derivative_at(f, x0, d) = forall(eps: Real) {
        eps.is_positive implies exists(delta: Real) {
            delta.is_positive and forall(x: Real) {
                x != x0 and x.is_close(x0, delta)
                implies difference_quotient(f, x0, x).is_close(d, eps)
            }
        }
    }
    forall(eps: Real) {
        if eps.is_positive {
            exists_small_mul_variant(c, eps)
            let eps2: Real satisfy {
                eps2.is_positive and c.abs * eps2 < eps
            }
            let delta: Real satisfy {
                delta.is_positive and forall(x: Real) {
                    x != x0 and x.is_close(x0, delta)
                    implies difference_quotient(f, x0, x).is_close(d, eps2)
                }
            }
            forall(x: Real) {
                if x != x0 and x.is_close(x0, delta) {
                    difference_quotient(f, x0, x).is_close(d, eps2)
                    scalar_mul_close(c, difference_quotient(f, x0, x), d, eps2, eps)
                    (c * difference_quotient(f, x0, x)).is_close(c * d, eps)
                    difference_quotient_pointwise_const_mul(c, f, x0, x)
                    difference_quotient(pointwise_mul(constant[Real, Real](c), f), x0, x) =
                        c * difference_quotient(f, x0, x)
                    difference_quotient(pointwise_mul(constant[Real, Real](c), f), x0, x).is_close(c * d, eps)
                }
            }
            exists(delta2: Real) {
                delta2.is_positive and forall(x: Real) {
                    x != x0 and x.is_close(x0, delta2)
                    implies difference_quotient(pointwise_mul(constant[Real, Real](c), f), x0, x).is_close(c * d, eps)
                }
            }
        }
    }
    has_derivative_at(pointwise_mul(constant[Real, Real](c), f), x0, c * d) = forall(eps: Real) {
        eps.is_positive implies exists(delta: Real) {
            delta.is_positive and forall(x: Real) {
                x != x0 and x.is_close(x0, delta)
                implies difference_quotient(pointwise_mul(constant[Real, Real](c), f), x0, x).is_close(c * d, eps)
            }
        }
    }
    if not has_derivative_at(pointwise_mul(constant[Real, Real](c), f), x0, c * d) {
        not forall(eps: Real) {
            eps.is_positive implies exists(delta: Real) {
                delta.is_positive and forall(x: Real) {
                    x != x0 and x.is_close(x0, delta)
                    implies difference_quotient(pointwise_mul(constant[Real, Real](c), f), x0, x).is_close(c * d, eps)
                }
            }
        }
        let bad_eps: Real satisfy {
            bad_eps.is_positive and forall(delta: Real) {
                not (delta.is_positive and forall(x: Real) {
                    x != x0 and x.is_close(x0, delta)
                    implies difference_quotient(pointwise_mul(constant[Real, Real](c), f), x0, x).is_close(c * d, bad_eps)
                })
            }
        }
        exists(delta: Real) {
            delta.is_positive and forall(x: Real) {
                x != x0 and x.is_close(x0, delta)
                implies difference_quotient(pointwise_mul(constant[Real, Real](c), f), x0, x).is_close(c * d, bad_eps)
            }
        }
        false
    }
}

/// Right scalar multiplication multiplies derivatives by that scalar.
theorem derivative_pointwise_mul_const(c: Real, f: Real -> Real, x0: Real, d: Real) {
    has_derivative_at(f, x0, d)
    implies has_derivative_at(pointwise_mul(f, constant[Real, Real](c)), x0, d * c)
} by {
    derivative_pointwise_const_mul(c, f, x0, d)
    pointwise_mul_comm[Real, Real](f, constant[Real, Real](c))
    pointwise_mul(f, constant[Real, Real](c)) = pointwise_mul(constant[Real, Real](c), f)
    c * d = d * c
    has_derivative_at(pointwise_mul(f, constant[Real, Real](c)), x0, d * c)
}

/// Differentiability is preserved by left scalar multiplication.
theorem differentiable_pointwise_const_mul(c: Real, f: Real -> Real, x0: Real) {
    differentiable_at(f, x0) implies differentiable_at(pointwise_mul(constant[Real, Real](c), f), x0)
} by {
    let d: Real satisfy {
        has_derivative_at(f, x0, d)
    }
    derivative_pointwise_const_mul(c, f, x0, d)
    has_derivative_at(pointwise_mul(constant[Real, Real](c), f), x0, c * d)
    exists(e: Real) {
        has_derivative_at(pointwise_mul(constant[Real, Real](c), f), x0, e)
    }
}

/// Differentiability is preserved by right scalar multiplication.
theorem differentiable_pointwise_mul_const(c: Real, f: Real -> Real, x0: Real) {
    differentiable_at(f, x0) implies differentiable_at(pointwise_mul(f, constant[Real, Real](c)), x0)
} by {
    let d: Real satisfy {
        has_derivative_at(f, x0, d)
    }
    derivative_pointwise_mul_const(c, f, x0, d)
    has_derivative_at(pointwise_mul(f, constant[Real, Real](c)), x0, d * c)
    exists(e: Real) {
        has_derivative_at(pointwise_mul(f, constant[Real, Real](c)), x0, e)
    }
}

/// Affine transformations of a differentiable function have the expected derivative.
theorem derivative_pointwise_affine(c: Real, b: Real, f: Real -> Real, x0: Real, d: Real) {
    has_derivative_at(f, x0, d)
    implies has_derivative_at(
        pointwise_add(pointwise_mul(constant[Real, Real](c), f), constant[Real, Real](b)),
        x0,
        c * d
    )
} by {
    derivative_pointwise_const_mul(c, f, x0, d)
    constant_has_derivative_at(b, x0)
    derivative_pointwise_add(pointwise_mul(constant[Real, Real](c), f), constant[Real, Real](b), x0, c * d, Real.0)
    has_derivative_at(
        pointwise_add(pointwise_mul(constant[Real, Real](c), f), constant[Real, Real](b)),
        x0,
        c * d + Real.0
    )
    c * d + Real.0 = c * d
    has_derivative_at(
        pointwise_add(pointwise_mul(constant[Real, Real](c), f), constant[Real, Real](b)),
        x0,
        c * d
    )
}

/// Differentiability is preserved by affine transformations of the function value.
theorem differentiable_pointwise_affine(c: Real, b: Real, f: Real -> Real, x0: Real) {
    differentiable_at(f, x0) implies differentiable_at(
        pointwise_add(pointwise_mul(constant[Real, Real](c), f), constant[Real, Real](b)),
        x0
    )
} by {
    let d: Real satisfy {
        has_derivative_at(f, x0, d)
    }
    derivative_pointwise_affine(c, b, f, x0, d)
    has_derivative_at(
        pointwise_add(pointwise_mul(constant[Real, Real](c), f), constant[Real, Real](b)),
        x0,
        c * d
    )
    exists(e: Real) {
        has_derivative_at(
            pointwise_add(pointwise_mul(constant[Real, Real](c), f), constant[Real, Real](b)),
            x0,
            e
        )
    }
}

/// Linear combinations of differentiable functions have the expected derivative.
theorem derivative_pointwise_linear_combination(
    a: Real, b: Real, f: Real -> Real, g: Real -> Real, x0: Real, df: Real, dg: Real
) {
    has_derivative_at(f, x0, df) and has_derivative_at(g, x0, dg)
    implies has_derivative_at(
        pointwise_add(pointwise_mul(constant[Real, Real](a), f), pointwise_mul(constant[Real, Real](b), g)),
        x0,
        a * df + b * dg
    )
} by {
    derivative_pointwise_const_mul(a, f, x0, df)
    derivative_pointwise_const_mul(b, g, x0, dg)
    has_derivative_at(pointwise_mul(constant[Real, Real](a), f), x0, a * df)
    has_derivative_at(pointwise_mul(constant[Real, Real](b), g), x0, b * dg)
    derivative_pointwise_add(
        pointwise_mul(constant[Real, Real](a), f),
        pointwise_mul(constant[Real, Real](b), g),
        x0,
        a * df,
        b * dg
    )
    has_derivative_at(
        pointwise_add(pointwise_mul(constant[Real, Real](a), f), pointwise_mul(constant[Real, Real](b), g)),
        x0,
        a * df + b * dg
    )
}

/// Differentiability is preserved by linear combinations.
theorem differentiable_pointwise_linear_combination(
    a: Real, b: Real, f: Real -> Real, g: Real -> Real, x0: Real
) {
    differentiable_at(f, x0) and differentiable_at(g, x0)
    implies differentiable_at(
        pointwise_add(pointwise_mul(constant[Real, Real](a), f), pointwise_mul(constant[Real, Real](b), g)),
        x0
    )
} by {
    let df: Real satisfy {
        has_derivative_at(f, x0, df)
    }
    let dg: Real satisfy {
        has_derivative_at(g, x0, dg)
    }
    derivative_pointwise_linear_combination(a, b, f, g, x0, df, dg)
    has_derivative_at(
        pointwise_add(pointwise_mul(constant[Real, Real](a), f), pointwise_mul(constant[Real, Real](b), g)),
        x0,
        a * df + b * dg
    )
    exists(d: Real) {
        has_derivative_at(
            pointwise_add(pointwise_mul(constant[Real, Real](a), f), pointwise_mul(constant[Real, Real](b), g)),
            x0,
            d
        )
    }
}

/// Real affine functions have derivative equal to their slope at every point.
theorem affine_identity_has_derivative_at(c: Real, b: Real, x0: Real) {
    has_derivative_at(
        pointwise_add(pointwise_mul(constant[Real, Real](c), identity_fn[Real]), constant[Real, Real](b)),
        x0,
        c
    )
} by {
    identity_has_derivative_at(x0)
    derivative_pointwise_affine(c, b, identity_fn[Real], x0, Real.1)
    c * Real.1 = c
    has_derivative_at(
        pointwise_add(pointwise_mul(constant[Real, Real](c), identity_fn[Real]), constant[Real, Real](b)),
        x0,
        c
    )
}

/// Real affine functions are differentiable at every point.
theorem affine_identity_differentiable_at(c: Real, b: Real, x0: Real) {
    differentiable_at(
        pointwise_add(pointwise_mul(constant[Real, Real](c), identity_fn[Real]), constant[Real, Real](b)),
        x0
    )
} by {
    affine_identity_has_derivative_at(c, b, x0)
    exists(d: Real) {
        has_derivative_at(
            pointwise_add(pointwise_mul(constant[Real, Real](c), identity_fn[Real]), constant[Real, Real](b)),
            x0,
            d
        )
    }
}
