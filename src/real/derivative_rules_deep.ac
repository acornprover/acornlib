/// Deep derivative rules: simplified derivatives of the square and cube
/// functions, the power rule for x^(n+1), restatements of the product rule
/// and the chain rule, and the exponential derivative.

from data.basic.function_algebra import pointwise_add, pointwise_mul
from data.basic.functions import compose, identity_fn
from nat import Nat, alt_induction, from_nat, from_nat_add, pow_zero
from real.calculus_api import derivative_fn_compose, derivative_fn_mul, is_derivative_fn
from real.continuity_cube import cube_real
from real.continuity_pow import pow_real_fn, pow_real_fn_suc, pow_real_fn_zero
from real.continuity_square import square_real
from real.derivative_basic import constant_has_derivative_at, differentiable_at, has_derivative_at, identity_has_derivative_at
from real.derivative_chain import derivative_compose
from real.derivative_exp_log import exp_has_derivative_at, exp_is_derivative_fn
from real.derivative_polynomial_chain import cube_real_has_derivative_at, square_real_has_derivative_at
from real.derivative_product import derivative_pointwise_mul
from real.exp import pow_suc, three, two
from real.real_base import Real

/// A derivative statement is preserved when the derivative value is replaced
/// by an equal value.
theorem has_derivative_at_eq_right(f: Real -> Real, x0: Real, d1: Real, d2: Real) {
    has_derivative_at(f, x0, d1) and d1 = d2 implies has_derivative_at(f, x0, d2)
} by {
    if has_derivative_at(f, x0, d1) and d1 = d2 {
        has_derivative_at(f, x0, d2)
    }
}

/// The square function has derivative 2x at every point.
theorem square_real_has_derivative_at_two_x(x0: Real) {
    has_derivative_at(square_real, x0, two * x0)
} by {
    square_real_has_derivative_at(x0)
    has_derivative_at(square_real, x0, x0 * Real.1 + x0 * Real.1)
    x0 * Real.1 + x0 * Real.1 = two * x0
    has_derivative_at_eq_right(square_real, x0, x0 * Real.1 + x0 * Real.1, two * x0)
    has_derivative_at(square_real, x0, two * x0)
}

/// The square function is differentiable at every point, with derivative 2x.
theorem square_real_differentiable_at_any(x0: Real) {
    differentiable_at(square_real, x0)
} by {
    square_real_has_derivative_at_two_x(x0)
    has_derivative_at(square_real, x0, two * x0)
    exists(d: Real) {
        has_derivative_at(square_real, x0, d)
    }
}

/// Two times a real is two copies of it.
theorem two_mul_real(e: Real) {
    two * e = e + e
} by {
}

/// Three times a real is three copies of it.
theorem three_mul_real(e: Real) {
    three * e = e + e + e
} by {
}

/// Three times a square is three copies of the square.
theorem three_mul_square(x0: Real) {
    three * (x0 * x0) = x0 * x0 + x0 * x0 + x0 * x0
} by {
}

/// The product-rule derivative value of the cube function simplifies to 3x^2.
theorem cube_derivative_value_three_square(x0: Real) {
    square_real(x0) * Real.1 + x0 * (x0 * Real.1 + x0 * Real.1) = three * square_real(x0)
} by {
    square_real(x0) = x0 * x0
    x0 * x0 * Real.1 = x0 * x0
    x0 * (x0 * Real.1 + x0 * Real.1) = x0 * (x0 + x0)
    x0 * (x0 + x0) = x0 * x0 + x0 * x0
    x0 * x0 * Real.1 + x0 * (x0 * Real.1 + x0 * Real.1) =
        x0 * x0 + (x0 * x0 + x0 * x0)
    x0 * x0 + (x0 * x0 + x0 * x0) = x0 * x0 + x0 * x0 + x0 * x0
    three_mul_square(x0)
    three * (x0 * x0) = x0 * x0 + x0 * x0 + x0 * x0
    three * (x0 * x0) = three * square_real(x0)
    square_real(x0) * Real.1 + x0 * (x0 * Real.1 + x0 * Real.1) = three * square_real(x0)
}

/// The cube function has derivative 3x^2 at every point.
theorem cube_real_has_derivative_at_three_square(x0: Real) {
    has_derivative_at(cube_real, x0, three * square_real(x0))
} by {
    cube_real_has_derivative_at(x0)
    has_derivative_at(
        cube_real,
        x0,
        square_real(x0) * Real.1 + x0 * (x0 * Real.1 + x0 * Real.1)
    )
    cube_derivative_value_three_square(x0)
    square_real(x0) * Real.1 + x0 * (x0 * Real.1 + x0 * Real.1) = three * square_real(x0)
    has_derivative_at_eq_right(
        cube_real,
        x0,
        square_real(x0) * Real.1 + x0 * (x0 * Real.1 + x0 * Real.1),
        three * square_real(x0)
    )
    has_derivative_at(cube_real, x0, three * square_real(x0))
}

/// The cube function is differentiable at every point, with derivative 3x^2.
theorem cube_real_differentiable_at_any(x0: Real) {
    differentiable_at(cube_real, x0)
} by {
    cube_real_has_derivative_at_three_square(x0)
    has_derivative_at(cube_real, x0, three * square_real(x0))
    exists(d: Real) {
        has_derivative_at(cube_real, x0, d)
    }
}

/// The product-rule value for the successor power simplifies to (k+2)x^(k+1).
theorem pow_real_fn_derivative_value_suc(k: Nat, x0: Real) {
    x0 * (from_nat[Real](k.suc) * x0.pow(k)) + x0 * x0.pow(k) =
        from_nat[Real](k.suc.suc) * x0.pow(k.suc)
} by {
    pow_suc(x0, k)
    x0.pow(k.suc) = x0 * x0.pow(k)
    from_nat_add[Real](k.suc, Nat.1)
    from_nat[Real](k.suc + Nat.1) = from_nat[Real](k.suc) + from_nat[Real](Nat.1)
    k.suc + Nat.1 = k.suc.suc
    from_nat[Real](k.suc.suc) = from_nat[Real](k.suc) + from_nat[Real](Nat.1)
    from_nat[Real](Nat.1) = Real.1
    from_nat[Real](k.suc.suc) = from_nat[Real](k.suc) + Real.1
    x0 * (from_nat[Real](k.suc) * x0.pow(k)) + x0 * x0.pow(k) =
        from_nat[Real](k.suc) * (x0 * x0.pow(k)) + x0 * x0.pow(k)
    from_nat[Real](k.suc) * (x0 * x0.pow(k)) + x0 * x0.pow(k) =
        from_nat[Real](k.suc) * x0.pow(k.suc) + x0.pow(k.suc)
    from_nat[Real](k.suc) * x0.pow(k.suc) + x0.pow(k.suc) =
        (from_nat[Real](k.suc) + Real.1) * x0.pow(k.suc)
    (from_nat[Real](k.suc) + Real.1) * x0.pow(k.suc) =
        from_nat[Real](k.suc.suc) * x0.pow(k.suc)
}

/// The first power function has derivative one at every point.
theorem pow_real_fn_has_derivative_at_base(x0: Real) {
    has_derivative_at(pow_real_fn(Nat.1), x0, from_nat[Real](Nat.1) * x0.pow(Nat.0))
} by {
    pow_real_fn_zero
    pow_real_fn(Nat.0) = constant[Real, Real](Real.1)
    constant_has_derivative_at(Real.1, x0)
    has_derivative_at(constant[Real, Real](Real.1), x0, Real.0)
    has_derivative_at(pow_real_fn(Nat.0), x0, Real.0)
    identity_has_derivative_at(x0)
    has_derivative_at(identity_fn[Real], x0, Real.1)
    derivative_pointwise_mul(identity_fn[Real], pow_real_fn(Nat.0), x0, Real.1, Real.0)
    has_derivative_at(pointwise_mul[Real, Real](identity_fn[Real], pow_real_fn(Nat.0)), x0,
        identity_fn[Real](x0) * Real.0 + pow_real_fn(Nat.0, x0) * Real.1)
    pow_real_fn_suc(Nat.0)
    pow_real_fn(Nat.1) = pointwise_mul[Real, Real](identity_fn[Real], pow_real_fn(Nat.0))
    has_derivative_at(pow_real_fn(Nat.1), x0,
        identity_fn[Real](x0) * Real.0 + pow_real_fn(Nat.0, x0) * Real.1)
    identity_fn[Real](x0) = x0
    pow_real_fn(Nat.0, x0) = Real.1
    identity_fn[Real](x0) * Real.0 + pow_real_fn(Nat.0, x0) * Real.1 =
        from_nat[Real](Nat.1) * x0.pow(Nat.0)
    has_derivative_at_eq_right(pow_real_fn(Nat.1), x0,
        identity_fn[Real](x0) * Real.0 + pow_real_fn(Nat.0, x0) * Real.1,
        from_nat[Real](Nat.1) * x0.pow(Nat.0))
    has_derivative_at(pow_real_fn(Nat.1), x0, from_nat[Real](Nat.1) * x0.pow(Nat.0))
}

/// The power rule induction step: the derivative of x^(k+2) follows from the
/// derivative of x^(k+1) by the product rule.
theorem pow_real_fn_has_derivative_at_suc(k: Nat, x0: Real) {
    has_derivative_at(pow_real_fn(k.suc), x0, from_nat[Real](k.suc) * x0.pow(k))
    implies has_derivative_at(pow_real_fn(k.suc.suc), x0, from_nat[Real](k.suc.suc) * x0.pow(k.suc))
} by {
    if has_derivative_at(pow_real_fn(k.suc), x0, from_nat[Real](k.suc) * x0.pow(k)) {
        identity_has_derivative_at(x0)
        has_derivative_at(identity_fn[Real], x0, Real.1)
        derivative_pointwise_mul(identity_fn[Real], pow_real_fn(k.suc), x0, Real.1,
            from_nat[Real](k.suc) * x0.pow(k))
        has_derivative_at(pointwise_mul[Real, Real](identity_fn[Real], pow_real_fn(k.suc)), x0,
            identity_fn[Real](x0) * (from_nat[Real](k.suc) * x0.pow(k)) +
            pow_real_fn(k.suc, x0) * Real.1)
        pow_real_fn_suc(k.suc)
        pow_real_fn(k.suc.suc) = pointwise_mul[Real, Real](identity_fn[Real], pow_real_fn(k.suc))
        has_derivative_at(pow_real_fn(k.suc.suc), x0,
            identity_fn[Real](x0) * (from_nat[Real](k.suc) * x0.pow(k)) +
            pow_real_fn(k.suc, x0) * Real.1)
        identity_fn[Real](x0) = x0
        pow_real_fn(k.suc, x0) = x0.pow(k.suc)
        pow_suc(x0, k)
        x0.pow(k.suc) = x0 * x0.pow(k)
        identity_fn[Real](x0) * (from_nat[Real](k.suc) * x0.pow(k)) +
            pow_real_fn(k.suc, x0) * Real.1 =
            x0 * (from_nat[Real](k.suc) * x0.pow(k)) + x0.pow(k.suc) * Real.1
        x0 * (from_nat[Real](k.suc) * x0.pow(k)) + x0.pow(k.suc) * Real.1 =
            x0 * (from_nat[Real](k.suc) * x0.pow(k)) + x0 * x0.pow(k)
        pow_real_fn_derivative_value_suc(k, x0)
        x0 * (from_nat[Real](k.suc) * x0.pow(k)) + x0 * x0.pow(k) =
            from_nat[Real](k.suc.suc) * x0.pow(k.suc)
        x0 * (from_nat[Real](k.suc) * x0.pow(k)) + x0.pow(k.suc) * Real.1 =
            from_nat[Real](k.suc.suc) * x0.pow(k.suc)
        identity_fn[Real](x0) * (from_nat[Real](k.suc) * x0.pow(k)) +
            pow_real_fn(k.suc, x0) * Real.1 =
            from_nat[Real](k.suc.suc) * x0.pow(k.suc)
        has_derivative_at_eq_right(pow_real_fn(k.suc.suc), x0,
            identity_fn[Real](x0) * (from_nat[Real](k.suc) * x0.pow(k)) +
            pow_real_fn(k.suc, x0) * Real.1,
            from_nat[Real](k.suc.suc) * x0.pow(k.suc))
        has_derivative_at(pow_real_fn(k.suc.suc), x0, from_nat[Real](k.suc.suc) * x0.pow(k.suc))
    }
}

/// The power rule: the derivative of x^(n+1) is (n+1) x^n at every point.
theorem pow_real_fn_has_derivative_at(n: Nat, x0: Real) {
    has_derivative_at(pow_real_fn(n.suc), x0, from_nat[Real](n.suc) * x0.pow(n))
} by {
    define p(k: Nat) -> Bool {
        has_derivative_at(pow_real_fn(k.suc), x0, from_nat[Real](k.suc) * x0.pow(k))
    }
    pow_real_fn_has_derivative_at_base(x0)
    p(Nat.0)
    forall(k: Nat) {
        if p(k) {
            pow_real_fn_has_derivative_at_suc(k, x0)
            has_derivative_at(pow_real_fn(k.suc.suc), x0, from_nat[Real](k.suc.suc) * x0.pow(k.suc))
            p(k.suc)
        }
    }
    p(Nat.0) and forall(x: Nat) { p(x) implies p(x.suc) }
    alt_induction(p)
    forall(x: Nat) { p(x) }
    p(n)
}

/// Every power function x^(n+1) is differentiable at every point.
theorem pow_real_fn_differentiable_at(n: Nat, x0: Real) {
    differentiable_at(pow_real_fn(n.suc), x0)
} by {
    pow_real_fn_has_derivative_at(n, x0)
    has_derivative_at(pow_real_fn(n.suc), x0, from_nat[Real](n.suc) * x0.pow(n))
    exists(d: Real) {
        has_derivative_at(pow_real_fn(n.suc), x0, d)
    }
}

/// The product rule for derivatives at a point:
/// (fg)'(x0) = f(x0) g'(x0) + g(x0) f'(x0).
theorem derivative_pointwise_mul_restated(f: Real -> Real, g: Real -> Real, x0: Real, df: Real, dg: Real) {
    has_derivative_at(f, x0, df) and has_derivative_at(g, x0, dg)
    implies has_derivative_at(pointwise_mul(f, g), x0, f(x0) * dg + g(x0) * df)
} by {
    derivative_pointwise_mul(f, g, x0, df, dg)
    has_derivative_at(pointwise_mul(f, g), x0, f(x0) * dg + g(x0) * df)
}

/// The product rule for global derivative functions:
/// (fg)' = f g' + g f'.
theorem derivative_fn_mul_restated(f: Real -> Real, g: Real -> Real, df: Real -> Real, dg: Real -> Real) {
    is_derivative_fn(f, df) and is_derivative_fn(g, dg)
    implies is_derivative_fn(
        pointwise_mul(f, g),
        pointwise_add(pointwise_mul(f, dg), pointwise_mul(g, df))
    )
} by {
    derivative_fn_mul(f, g, df, dg)
    is_derivative_fn(
        pointwise_mul(f, g),
        pointwise_add(pointwise_mul(f, dg), pointwise_mul(g, df))
    )
}

/// The chain rule for derivatives at a point:
/// (g ∘ f)'(x0) = g'(f(x0)) f'(x0).
theorem derivative_compose_restated(f: Real -> Real, g: Real -> Real, x0: Real, df: Real, dg: Real) {
    has_derivative_at(f, x0, df) and has_derivative_at(g, f(x0), dg)
    implies has_derivative_at(compose(g, f), x0, dg * df)
} by {
    derivative_compose(f, g, x0, df, dg)
    has_derivative_at(compose(g, f), x0, dg * df)
}

/// The chain rule for global derivative functions:
/// (g ∘ f)' = (g' ∘ f) · f'.
theorem derivative_fn_compose_restated(f: Real -> Real, g: Real -> Real, df: Real -> Real, dg: Real -> Real) {
    is_derivative_fn(f, df) and is_derivative_fn(g, dg)
    implies is_derivative_fn(compose(g, f), pointwise_mul(compose(dg, f), df))
} by {
    derivative_fn_compose(f, g, df, dg)
    is_derivative_fn(compose(g, f), pointwise_mul(compose(dg, f), df))
}

/// The exponential function has derivative e^x at every point.
theorem exp_has_derivative_at_self(x0: Real) {
    has_derivative_at(Real.exp, x0, x0.exp)
} by {
    exp_has_derivative_at(x0)
    has_derivative_at(Real.exp, x0, x0.exp)
}

/// The exponential function is its own global derivative function.
theorem exp_is_derivative_fn_self {
    is_derivative_fn(Real.exp, Real.exp)
} by {
    exp_is_derivative_fn
    is_derivative_fn(Real.exp, Real.exp)
}
