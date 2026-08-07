from data.basic.functions import compose
from real.continuity_cube import cube_real
from real.continuity_square import square_real
from real.derivative_basic import has_derivative_at, differentiable_at
from real.derivative_polynomial_chain import cube_real_differentiable_at,
    cube_real_has_derivative_at, derivative_cube_real_compose,
    derivative_square_real_compose, differentiable_cube_real_compose,
    differentiable_square_real_compose, square_real_differentiable_at,
    square_real_has_derivative_at
from real.real_base import Real

/// The square function after the square function follows the chain rule.
theorem derivative_square_real_after_square_real(x0: Real) {
    has_derivative_at(
        compose(square_real, square_real),
        x0,
        (square_real(x0) * Real.1 + square_real(x0) * Real.1) * (x0 * Real.1 + x0 * Real.1)
    )
} by {
    square_real_has_derivative_at(x0)
    derivative_square_real_compose(square_real, x0, x0 * Real.1 + x0 * Real.1)
    has_derivative_at(
        compose(square_real, square_real),
        x0,
        (square_real(x0) * Real.1 + square_real(x0) * Real.1) * (x0 * Real.1 + x0 * Real.1)
    )
}

/// The square function after the cube function follows the chain rule.
theorem derivative_square_real_after_cube_real(x0: Real) {
    has_derivative_at(
        compose(square_real, cube_real),
        x0,
        (cube_real(x0) * Real.1 + cube_real(x0) * Real.1) *
        (square_real(x0) * Real.1 + x0 * (x0 * Real.1 + x0 * Real.1))
    )
} by {
    cube_real_has_derivative_at(x0)
    derivative_square_real_compose(cube_real, x0,
        square_real(x0) * Real.1 + x0 * (x0 * Real.1 + x0 * Real.1))
    has_derivative_at(
        compose(square_real, cube_real),
        x0,
        (cube_real(x0) * Real.1 + cube_real(x0) * Real.1) *
        (square_real(x0) * Real.1 + x0 * (x0 * Real.1 + x0 * Real.1))
    )
}

/// The cube function after the square function follows the chain rule.
theorem derivative_cube_real_after_square_real(x0: Real) {
    has_derivative_at(
        compose(cube_real, square_real),
        x0,
        (
            square_real(square_real(x0)) * Real.1 +
            square_real(x0) * (square_real(x0) * Real.1 + square_real(x0) * Real.1)
        ) * (x0 * Real.1 + x0 * Real.1)
    )
} by {
    square_real_has_derivative_at(x0)
    derivative_cube_real_compose(square_real, x0, x0 * Real.1 + x0 * Real.1)
    has_derivative_at(
        compose(cube_real, square_real),
        x0,
        (
            square_real(square_real(x0)) * Real.1 +
            square_real(x0) * (square_real(x0) * Real.1 + square_real(x0) * Real.1)
        ) * (x0 * Real.1 + x0 * Real.1)
    )
}

/// The cube function after the cube function follows the chain rule.
theorem derivative_cube_real_after_cube_real(x0: Real) {
    has_derivative_at(
        compose(cube_real, cube_real),
        x0,
        (
            square_real(cube_real(x0)) * Real.1 +
            cube_real(x0) * (cube_real(x0) * Real.1 + cube_real(x0) * Real.1)
        ) * (square_real(x0) * Real.1 + x0 * (x0 * Real.1 + x0 * Real.1))
    )
} by {
    cube_real_has_derivative_at(x0)
    derivative_cube_real_compose(cube_real, x0,
        square_real(x0) * Real.1 + x0 * (x0 * Real.1 + x0 * Real.1))
    has_derivative_at(
        compose(cube_real, cube_real),
        x0,
        (
            square_real(cube_real(x0)) * Real.1 +
            cube_real(x0) * (cube_real(x0) * Real.1 + cube_real(x0) * Real.1)
        ) * (square_real(x0) * Real.1 + x0 * (x0 * Real.1 + x0 * Real.1))
    )
}

/// The square function after the square function is differentiable.
theorem differentiable_square_real_after_square_real(x0: Real) {
    differentiable_at(compose(square_real, square_real), x0)
} by {
    square_real_differentiable_at(x0)
    differentiable_square_real_compose(square_real, x0)
    differentiable_at(compose(square_real, square_real), x0)
}

/// The square function after the cube function is differentiable.
theorem differentiable_square_real_after_cube_real(x0: Real) {
    differentiable_at(compose(square_real, cube_real), x0)
} by {
    cube_real_differentiable_at(x0)
    differentiable_square_real_compose(cube_real, x0)
    differentiable_at(compose(square_real, cube_real), x0)
}

/// The cube function after the square function is differentiable.
theorem differentiable_cube_real_after_square_real(x0: Real) {
    differentiable_at(compose(cube_real, square_real), x0)
} by {
    square_real_differentiable_at(x0)
    differentiable_cube_real_compose(square_real, x0)
    differentiable_at(compose(cube_real, square_real), x0)
}

/// The cube function after the cube function is differentiable.
theorem differentiable_cube_real_after_cube_real(x0: Real) {
    differentiable_at(compose(cube_real, cube_real), x0)
} by {
    cube_real_differentiable_at(x0)
    differentiable_cube_real_compose(cube_real, x0)
    differentiable_at(compose(cube_real, cube_real), x0)
}
