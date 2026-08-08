from nat import Nat, lte_trans
from list import List, partial
from list import partial_add, partial_pointwise_eq, reverse_index, partial_reverse
from data.basic.functions import flip
from algebra.add_semigroup import add_fn
from real.real_series import Real, is_lower_bound, partial_nonneg, const_seq, const_seq_n, is_increasing, distant_increasing, increasing_is_monotone
from order import is_monotone
from real.rectangular_sum import nat_to_real, nonneg_fn_2, is_upper_bound_fn_2, is_lower_bound_fn_2, lte_fn_2, const_fn_2, add_fn_2, sub_fn_2, scalar_mul_fn_2, rectangular_sum, row_sum, shift_rows, shift_cols
from real.double_sum import square_sum, double_sum_converges, double_sum, square_sum_converges_to_double_sum, pos_part_2, neg_part_2, abs_fn_2, double_image_is_supremum, double_sum_pos_part_converges_of_supremum, double_sum_neg_part_converges_of_supremum, row_iterated_limit_eq_double_sum, limit_sub_seq, rectangular_sum_decomposition, sub_seq_eq_add_neg, partial_sub_seq
from real.double_limit import double_limit_sub, double_converges, double_limit, double_converges_to, double_limit_eq_of_converges_to
from real.abs_conv import sub_seq
from real.real_seq import converges, limit, converges_to, tail_bound, converges_to_imp_converges, converges_imp_converges_to, add_seq
from real.real_series import add_seq_converges, neg_seq, neg_seq_converges

numerals Real
numerals Nat

/// This file defines triangular sums and proves theorems about them.
/// A triangular sum sums f(i, j) over all pairs where i + j < n.

/// Sum of f(i, j) for j < n - i, which gives j where i + j < n.
define tri_row_sum(f: (Nat, Nat) -> Real, n: Nat, i: Nat) -> Real {
    partial(f(i), n - i)
}

/// The diagonal element with total index n at position i.
define diagonal(f: (Nat, Nat) -> Real, n: Nat, i: Nat) -> Real {
    f(i, n - i)
}

/// Sum of diagonal elements f(i, n - i) for i <= n.
define diagonal_sum(f: (Nat, Nat) -> Real, n: Nat) -> Real {
    partial(diagonal(f, n), n.suc)
}

/// The triangular sum of f over pairs (i, j) where i + j < n.
/// Computes sum_{i+j < n} f(i, j).
define triangular_sum(f: (Nat, Nat) -> Real, n: Nat) -> Real {
    partial(tri_row_sum(f, n), n)
}

/// Triangular sums as a sequence in the truncation bound.
define triangular_sum_seq(f: (Nat, Nat) -> Real, n: Nat) -> Real {
    triangular_sum(f, n)
}

/// Triangular sum of zero is zero.
theorem triangular_sum_zero(f: (Nat, Nat) -> Real) {
    triangular_sum(f, Nat.0) = Real.0
}

/// Triangular sum is nonnegative for nonnegative functions.
theorem triangular_sum_nonneg(f: (Nat, Nat) -> Real, n: Nat) {
    nonneg_fn_2(f)
    implies
    triangular_sum(f, n) >= Real.0
} by {
    define p(k: Nat) -> Bool {
        nonneg_fn_2(f)
        implies
        triangular_sum(f, k) >= Real.0
    }

    p(Nat.0)

    forall(k: Nat) {
        if p(k) {
            if nonneg_fn_2(f) {
                // Prove tri_row_sum is nonneg for all i
                forall(i: Nat) {
                    // Prove partial(f(i), m) >= 0 for any m
                    define s(m: Nat) -> Bool {
                        partial(f(i), m) >= Real.0
                    }

                    s(Nat.0)

                    forall(m: Nat) {
                        if s(m) {
                            f(i, m) >= Real.0
                            partial(f(i), m.suc) = partial(f(i), m) + f(i, m)
                            Real.0 + Real.0 <= partial(f(i), m) + f(i, m)
                            partial(f(i), m.suc) >= Real.0
                            s(m.suc)
                        }
                    }

                    s(k.suc - i)
                    tri_row_sum(f, k.suc, i) = partial(f(i), k.suc - i)
                    tri_row_sum(f, k.suc, i) >= Real.0
                }

                // Now prove triangular_sum(f, k.suc) >= 0
                define t(m: Nat) -> Bool {
                    partial(tri_row_sum(f, k.suc), m) >= Real.0
                }

                t(Nat.0)

                forall(m: Nat) {
                    if t(m) {
                        tri_row_sum(f, k.suc, m) >= Real.0
                        partial(tri_row_sum(f, k.suc), m.suc) = partial(tri_row_sum(f, k.suc), m) + tri_row_sum(f, k.suc, m)
                        Real.0 + Real.0 <= partial(tri_row_sum(f, k.suc), m) + tri_row_sum(f, k.suc, m)
                        partial(tri_row_sum(f, k.suc), m.suc) >= Real.0
                        t(m.suc)
                    }
                }

                t(k.suc)
                triangular_sum(f, k.suc) = partial(tri_row_sum(f, k.suc), k.suc)
                triangular_sum(f, k.suc) >= Real.0
            }
            p(k.suc)
        }
    }
    p(n)
}

/// Triangular sums satisfy the recurrence adding the new diagonal.
theorem triangular_sum_step(f: (Nat, Nat) -> Real, n: Nat) {
    triangular_sum(f, n.suc) = triangular_sum(f, n) + diagonal_sum(f, n)
} by {
    // Expand the definitions of the sums.
    triangular_sum(f, n.suc) = partial(tri_row_sum(f, n.suc), n.suc)
    triangular_sum(f, n) = partial(tri_row_sum(f, n), n)
    diagonal_sum(f, n) = partial(diagonal(f, n), n.suc)

    // For i < n, the row sums differ by the diagonal element.
    forall(i: Nat) {
        if i < n {
            i < n.suc
            n >= i
            i <= n
            n - i + i = n
            (n - i + i).suc = (n - i).suc + i
            (n - i).suc + i = n.suc
            n.suc >= i
            n.suc - i = (n - i).suc
            tri_row_sum(f, n.suc, i) = partial(f(i), n.suc - i)
            tri_row_sum(f, n.suc, i) = partial(f(i), (n - i).suc)
            tri_row_sum(f, n, i) = partial(f(i), n - i)
            partial(f(i), (n - i).suc) = partial(f(i), n - i) + f(i, n - i)
            tri_row_sum(f, n.suc, i) = tri_row_sum(f, n, i) + f(i, n - i)
            diagonal(f, n, i) = f(i, n - i)
            tri_row_sum(f, n.suc, i) = tri_row_sum(f, n, i) + diagonal(f, n, i)
            add_fn(tri_row_sum(f, n), diagonal(f, n), i) = tri_row_sum(f, n, i) + diagonal(f, n, i)
            tri_row_sum(f, n.suc, i) = add_fn(tri_row_sum(f, n), diagonal(f, n), i)
        }
    }

    partial_pointwise_eq(tri_row_sum(f, n.suc), add_fn(tri_row_sum(f, n), diagonal(f, n)), n)
    partial(tri_row_sum(f, n.suc), n) = partial(add_fn(tri_row_sum(f, n), diagonal(f, n)), n)

    partial(add_fn(tri_row_sum(f, n), diagonal(f, n)), n) = partial(tri_row_sum(f, n), n) + partial(diagonal(f, n), n)

    partial(tri_row_sum(f, n.suc), n) = partial(tri_row_sum(f, n), n) + partial(diagonal(f, n), n)

    partial(tri_row_sum(f, n.suc), n.suc) = partial(tri_row_sum(f, n.suc), n) + tri_row_sum(f, n.suc, n)
    n.suc - n = Nat.1
    tri_row_sum(f, n.suc, n) = partial(f(n), n.suc - n)
    tri_row_sum(f, n.suc, n) = partial(f(n), Nat.1)
    partial(f(n), Nat.1) = partial(f(n), Nat.0) + f(n, Nat.0)
    partial(f(n), Nat.0) = Real.0
    tri_row_sum(f, n.suc, n) = f(n, Nat.0)
    n - n = Nat.0
    diagonal(f, n, n) = f(n, n - n)
    diagonal(f, n, n) = f(n, Nat.0)
    tri_row_sum(f, n.suc, n) = diagonal(f, n, n)

    partial(diagonal(f, n), n.suc) = partial(diagonal(f, n), n) + diagonal(f, n, n)

    triangular_sum(f, n.suc) = partial(tri_row_sum(f, n), n) + partial(diagonal(f, n), n) + diagonal(f, n, n)
    triangular_sum(f, n) = partial(tri_row_sum(f, n), n)
    diagonal_sum(f, n) = partial(diagonal(f, n), n) + diagonal(f, n, n)
    triangular_sum(f, n.suc) = triangular_sum(f, n) + diagonal_sum(f, n)
}

/// If f <= g pointwise, then their triangular sums satisfy the same inequality.
theorem triangular_sum_monotone(f: (Nat, Nat) -> Real, g: (Nat, Nat) -> Real, n: Nat) {
    lte_fn_2(f, g)
    implies
    triangular_sum(f, n) <= triangular_sum(g, n)
} by {
    define p(k: Nat) -> Bool {
        lte_fn_2(f, g)
        implies
        triangular_sum(f, k) <= triangular_sum(g, k)
    }

    p(Nat.0)

    forall(k: Nat) {
        if p(k) {
            if lte_fn_2(f, g) {
                // Prove tri_row_sum(f, k.suc, i) <= tri_row_sum(g, k.suc, i) for all i
                forall(i: Nat) {
                    // Prove partial(f(i), m) <= partial(g(i), m) for any m
                    define s(m: Nat) -> Bool {
                        partial(f(i), m) <= partial(g(i), m)
                    }

                    s(Nat.0)

                    forall(m: Nat) {
                        if s(m) {
                            f(i, m) <= g(i, m)
                            partial(f(i), m) + f(i, m) <= partial(g(i), m) + g(i, m)
                            partial(f(i), m.suc) <= partial(g(i), m.suc)
                            s(m.suc)
                        }
                    }

                    partial(f(i), k.suc - i) <= partial(g(i), k.suc - i)
                    tri_row_sum(f, k.suc, i) = partial(f(i), k.suc - i)
                    tri_row_sum(g, k.suc, i) = partial(g(i), k.suc - i)
                    tri_row_sum(f, k.suc, i) <= tri_row_sum(g, k.suc, i)
                }

                // Now prove triangular_sum(f, k.suc) <= triangular_sum(g, k.suc)
                define t(m: Nat) -> Bool {
                    partial(tri_row_sum(f, k.suc), m) <= partial(tri_row_sum(g, k.suc), m)
                }

                t(Nat.0)

                forall(m: Nat) {
                    if t(m) {
                        tri_row_sum(f, k.suc, m) <= tri_row_sum(g, k.suc, m)
                        partial(tri_row_sum(f, k.suc), m) + tri_row_sum(f, k.suc, m) <= partial(tri_row_sum(g, k.suc), m) + tri_row_sum(g, k.suc, m)
                        partial(tri_row_sum(f, k.suc), m.suc) <= partial(tri_row_sum(g, k.suc), m.suc)
                        t(m.suc)
                    }
                }

                partial(tri_row_sum(f, k.suc), k.suc) <= partial(tri_row_sum(g, k.suc), k.suc)
                triangular_sum(f, k.suc) = partial(tri_row_sum(f, k.suc), k.suc)
                triangular_sum(g, k.suc) = partial(tri_row_sum(g, k.suc), k.suc)
                triangular_sum(f, k.suc) <= triangular_sum(g, k.suc)
            }
            p(k.suc)
        }
    }
    p(n)
}

/// Triangular sum is additive in the function.
theorem triangular_sum_add(f: (Nat, Nat) -> Real, g: (Nat, Nat) -> Real, n: Nat) {
    triangular_sum(add_fn_2(f, g), n)
    =
    triangular_sum(f, n) + triangular_sum(g, n)
} by {
    define p(k: Nat) -> Bool {
        triangular_sum(add_fn_2(f, g), k)
        =
        triangular_sum(f, k) + triangular_sum(g, k)
    }

    p(Nat.0)

    forall(k: Nat) {
        if p(k) {
            // Prove tri_row_sum additivity for all i
            forall(i: Nat) {
                // Prove partial additivity
                define s(m: Nat) -> Bool {
                    partial(add_fn_2(f, g)(i), m) = partial(f(i), m) + partial(g(i), m)
                }

                s(Nat.0)

                forall(m: Nat) {
                    if s(m) {
                        partial(add_fn_2(f, g)(i), m) = partial(f(i), m) + partial(g(i), m)
                        partial(add_fn_2(f, g)(i), m.suc) = partial(add_fn_2(f, g)(i), m) + add_fn_2(f, g)(i, m)
                        add_fn_2(f, g)(i, m) = f(i, m) + g(i, m)
                        partial(add_fn_2(f, g)(i), m.suc) = (partial(f(i), m) + partial(g(i), m)) + (f(i, m) + g(i, m))
                        (partial(f(i), m) + partial(g(i), m)) + (f(i, m) + g(i, m)) = partial(f(i), m) + partial(g(i), m) + f(i, m) + g(i, m)
                        partial(f(i), m) + partial(g(i), m) + f(i, m) = partial(f(i), m) + f(i, m) + partial(g(i), m)
                        partial(f(i), m) + partial(g(i), m) + f(i, m) + g(i, m) = partial(f(i), m) + f(i, m) + partial(g(i), m) + g(i, m)
                        partial(f(i), m) + f(i, m) + partial(g(i), m) + g(i, m) = (partial(f(i), m) + f(i, m)) + (partial(g(i), m) + g(i, m))
                        partial(add_fn_2(f, g)(i), m.suc) = (partial(f(i), m) + f(i, m)) + (partial(g(i), m) + g(i, m))
                        partial(f(i), m.suc) = partial(f(i), m) + f(i, m)
                        partial(g(i), m.suc) = partial(g(i), m) + g(i, m)
                        partial(add_fn_2(f, g)(i), m.suc) = partial(f(i), m.suc) + partial(g(i), m.suc)
                        s(m.suc)
                    }
                }

                s(k.suc - i)
                partial(add_fn_2(f, g)(i), k.suc - i) = partial(f(i), k.suc - i) + partial(g(i), k.suc - i)
                tri_row_sum(add_fn_2(f, g), k.suc, i) = partial(add_fn_2(f, g)(i), k.suc - i)
                tri_row_sum(f, k.suc, i) = partial(f(i), k.suc - i)
                tri_row_sum(g, k.suc, i) = partial(g(i), k.suc - i)
                tri_row_sum(add_fn_2(f, g), k.suc, i) = tri_row_sum(f, k.suc, i) + tri_row_sum(g, k.suc, i)
            }

            // Now prove triangular_sum additivity
            define t(m: Nat) -> Bool {
                partial(tri_row_sum(add_fn_2(f, g), k.suc), m) = partial(tri_row_sum(f, k.suc), m) + partial(tri_row_sum(g, k.suc), m)
            }

            t(Nat.0)

            forall(m: Nat) {
                if t(m) {
                    partial(tri_row_sum(f, k.suc), m.suc) = partial(tri_row_sum(f, k.suc), m) + tri_row_sum(f, k.suc, m)
                    partial(tri_row_sum(g, k.suc), m.suc) = partial(tri_row_sum(g, k.suc), m) + tri_row_sum(g, k.suc, m)
                    partial(tri_row_sum(f, k.suc), m.suc) + partial(tri_row_sum(g, k.suc), m.suc) = (partial(tri_row_sum(f, k.suc), m) + tri_row_sum(f, k.suc, m)) + (partial(tri_row_sum(g, k.suc), m) + tri_row_sum(g, k.suc, m))
                    tri_row_sum(add_fn_2(f, g), k.suc, m) = tri_row_sum(f, k.suc, m) + tri_row_sum(g, k.suc, m)
                    partial(tri_row_sum(add_fn_2(f, g), k.suc), m.suc) = partial(tri_row_sum(add_fn_2(f, g), k.suc), m) + tri_row_sum(add_fn_2(f, g), k.suc, m)
                    partial(tri_row_sum(add_fn_2(f, g), k.suc), m) = partial(tri_row_sum(f, k.suc), m) + partial(tri_row_sum(g, k.suc), m)
                    partial(tri_row_sum(add_fn_2(f, g), k.suc), m.suc) = (partial(tri_row_sum(f, k.suc), m) + partial(tri_row_sum(g, k.suc), m)) + (tri_row_sum(f, k.suc, m) + tri_row_sum(g, k.suc, m))
                    partial(tri_row_sum(add_fn_2(f, g), k.suc), m.suc) = (partial(tri_row_sum(f, k.suc), m) + tri_row_sum(f, k.suc, m)) + (partial(tri_row_sum(g, k.suc), m) + tri_row_sum(g, k.suc, m))
                    partial(tri_row_sum(add_fn_2(f, g), k.suc), m.suc) = partial(tri_row_sum(f, k.suc), m.suc) + partial(tri_row_sum(g, k.suc), m.suc)
                    t(m.suc)
                }
            }

            t(k.suc)
            partial(tri_row_sum(add_fn_2(f, g), k.suc), k.suc) = partial(tri_row_sum(f, k.suc), k.suc) + partial(tri_row_sum(g, k.suc), k.suc)
            triangular_sum(add_fn_2(f, g), k.suc) = partial(tri_row_sum(add_fn_2(f, g), k.suc), k.suc)
            triangular_sum(f, k.suc) = partial(tri_row_sum(f, k.suc), k.suc)
            triangular_sum(g, k.suc) = partial(tri_row_sum(g, k.suc), k.suc)
            triangular_sum(add_fn_2(f, g), k.suc) = triangular_sum(f, k.suc) + triangular_sum(g, k.suc)
            p(k.suc)
        }
    }
    p(n)
}

/// Scaling a triangular sum.
theorem triangular_sum_scale(c: Real, f: (Nat, Nat) -> Real, n: Nat) {
    triangular_sum(scalar_mul_fn_2(c, f), n)
    =
    c * triangular_sum(f, n)
} by {
    define p(k: Nat) -> Bool {
        triangular_sum(scalar_mul_fn_2(c, f), k)
        =
        c * triangular_sum(f, k)
    }

    p(Nat.0)

    forall(k: Nat) {
        if p(k) {
            // Prove tri_row_sum scales for all i
            forall(i: Nat) {
                // Prove partial scales
                define s(m: Nat) -> Bool {
                    partial(scalar_mul_fn_2(c, f)(i), m) = c * partial(f(i), m)
                }

                s(Nat.0)

                forall(m: Nat) {
                    if s(m) {
                        partial(scalar_mul_fn_2(c, f)(i), m) = c * partial(f(i), m)
                        partial(scalar_mul_fn_2(c, f)(i), m.suc) = partial(scalar_mul_fn_2(c, f)(i), m) + scalar_mul_fn_2(c, f)(i, m)
                        scalar_mul_fn_2(c, f)(i, m) = c * f(i, m)
                        partial(scalar_mul_fn_2(c, f)(i), m.suc) = c * partial(f(i), m) + c * f(i, m)
                        partial(f(i), m.suc) = partial(f(i), m) + f(i, m)
                        partial(scalar_mul_fn_2(c, f)(i), m.suc) = c * partial(f(i), m.suc)
                        s(m.suc)
                    }
                }

                s(k.suc - i)
                partial(scalar_mul_fn_2(c, f)(i), k.suc - i) = c * partial(f(i), k.suc - i)
                tri_row_sum(scalar_mul_fn_2(c, f), k.suc, i) = partial(scalar_mul_fn_2(c, f)(i), k.suc - i)
                tri_row_sum(f, k.suc, i) = partial(f(i), k.suc - i)
                tri_row_sum(scalar_mul_fn_2(c, f), k.suc, i) = c * tri_row_sum(f, k.suc, i)
            }

            // Now prove triangular_sum scales
            define t(m: Nat) -> Bool {
                partial(tri_row_sum(scalar_mul_fn_2(c, f), k.suc), m) = c * partial(tri_row_sum(f, k.suc), m)
            }

            t(Nat.0)

            forall(m: Nat) {
                if t(m) {
                    partial(tri_row_sum(scalar_mul_fn_2(c, f), k.suc), m) = c * partial(tri_row_sum(f, k.suc), m)
                    tri_row_sum(scalar_mul_fn_2(c, f), k.suc, m) = c * tri_row_sum(f, k.suc, m)
                    partial(tri_row_sum(scalar_mul_fn_2(c, f), k.suc), m.suc) = partial(tri_row_sum(scalar_mul_fn_2(c, f), k.suc), m) + tri_row_sum(scalar_mul_fn_2(c, f), k.suc, m)
                    partial(tri_row_sum(scalar_mul_fn_2(c, f), k.suc), m.suc) = c * partial(tri_row_sum(f, k.suc), m) + c * tri_row_sum(f, k.suc, m)
                    partial(tri_row_sum(f, k.suc), m.suc) = partial(tri_row_sum(f, k.suc), m) + tri_row_sum(f, k.suc, m)
                    partial(tri_row_sum(scalar_mul_fn_2(c, f), k.suc), m.suc) = c * partial(tri_row_sum(f, k.suc), m.suc)
                    t(m.suc)
                }
            }

            t(k.suc)
            partial(tri_row_sum(scalar_mul_fn_2(c, f), k.suc), k.suc) = c * partial(tri_row_sum(f, k.suc), k.suc)
            triangular_sum(scalar_mul_fn_2(c, f), k.suc) = partial(tri_row_sum(scalar_mul_fn_2(c, f), k.suc), k.suc)
            triangular_sum(f, k.suc) = partial(tri_row_sum(f, k.suc), k.suc)
            triangular_sum(scalar_mul_fn_2(c, f), k.suc) = c * triangular_sum(f, k.suc)
            p(k.suc)
        }
    }
    p(n)
}

/// Triangular sum is distributive over subtraction.
theorem triangular_sum_sub(f: (Nat, Nat) -> Real, g: (Nat, Nat) -> Real, n: Nat) {
    triangular_sum(sub_fn_2(f, g), n)
    =
    triangular_sum(f, n) - triangular_sum(g, n)
} by {
    define p(k: Nat) -> Bool {
        triangular_sum(sub_fn_2(f, g), k)
        =
        triangular_sum(f, k) - triangular_sum(g, k)
    }

    p(Nat.0)

    forall(k: Nat) {
        if p(k) {
            // Prove tri_row_sum subtractivity for all i
            forall(i: Nat) {
                // Prove partial subtractivity
                define s(m: Nat) -> Bool {
                    partial(sub_fn_2(f, g)(i), m) = partial(f(i), m) - partial(g(i), m)
                }

                s(Nat.0)

                forall(m: Nat) {
                    if s(m) {
                        partial(sub_fn_2(f, g)(i), m) = partial(f(i), m) - partial(g(i), m)
                        partial(sub_fn_2(f, g)(i), m.suc) = partial(sub_fn_2(f, g)(i), m) + sub_fn_2(f, g)(i, m)
                        sub_fn_2(f, g)(i, m) = f(i, m) - g(i, m)
                        partial(sub_fn_2(f, g)(i), m.suc) = (partial(f(i), m) - partial(g(i), m)) + (f(i, m) - g(i, m))
                        partial(sub_fn_2(f, g)(i), m.suc) = (partial(f(i), m) + f(i, m)) - (partial(g(i), m) + g(i, m))
                        partial(f(i), m.suc) = partial(f(i), m) + f(i, m)
                        partial(g(i), m.suc) = partial(g(i), m) + g(i, m)
                        partial(sub_fn_2(f, g)(i), m.suc) = partial(f(i), m.suc) - partial(g(i), m.suc)
                        s(m.suc)
                    }
                }

                s(k.suc - i)
                partial(sub_fn_2(f, g)(i), k.suc - i) = partial(f(i), k.suc - i) - partial(g(i), k.suc - i)
                tri_row_sum(sub_fn_2(f, g), k.suc, i) = partial(sub_fn_2(f, g)(i), k.suc - i)
                tri_row_sum(f, k.suc, i) = partial(f(i), k.suc - i)
                tri_row_sum(g, k.suc, i) = partial(g(i), k.suc - i)
                tri_row_sum(sub_fn_2(f, g), k.suc, i) = tri_row_sum(f, k.suc, i) - tri_row_sum(g, k.suc, i)
            }

            // Now prove triangular_sum subtractivity
            define t(m: Nat) -> Bool {
                partial(tri_row_sum(sub_fn_2(f, g), k.suc), m) = partial(tri_row_sum(f, k.suc), m) - partial(tri_row_sum(g, k.suc), m)
            }

            t(Nat.0)

            forall(m: Nat) {
                if t(m) {
                    partial(tri_row_sum(f, k.suc), m.suc) = partial(tri_row_sum(f, k.suc), m) + tri_row_sum(f, k.suc, m)
                    partial(tri_row_sum(g, k.suc), m.suc) = partial(tri_row_sum(g, k.suc), m) + tri_row_sum(g, k.suc, m)
                    partial(tri_row_sum(f, k.suc), m.suc) - partial(tri_row_sum(g, k.suc), m.suc) = (partial(tri_row_sum(f, k.suc), m) + tri_row_sum(f, k.suc, m)) - (partial(tri_row_sum(g, k.suc), m) + tri_row_sum(g, k.suc, m))
                    tri_row_sum(sub_fn_2(f, g), k.suc, m) = tri_row_sum(f, k.suc, m) - tri_row_sum(g, k.suc, m)
                    partial(tri_row_sum(sub_fn_2(f, g), k.suc), m.suc) = partial(tri_row_sum(sub_fn_2(f, g), k.suc), m) + tri_row_sum(sub_fn_2(f, g), k.suc, m)
                    partial(tri_row_sum(sub_fn_2(f, g), k.suc), m) = partial(tri_row_sum(f, k.suc), m) - partial(tri_row_sum(g, k.suc), m)
                    partial(tri_row_sum(sub_fn_2(f, g), k.suc), m.suc) = (partial(tri_row_sum(f, k.suc), m) - partial(tri_row_sum(g, k.suc), m)) + (tri_row_sum(f, k.suc, m) - tri_row_sum(g, k.suc, m))
                    partial(tri_row_sum(sub_fn_2(f, g), k.suc), m.suc) = (partial(tri_row_sum(f, k.suc), m) + tri_row_sum(f, k.suc, m)) - (partial(tri_row_sum(g, k.suc), m) + tri_row_sum(g, k.suc, m))
                    partial(tri_row_sum(sub_fn_2(f, g), k.suc), m.suc) = partial(tri_row_sum(f, k.suc), m.suc) - partial(tri_row_sum(g, k.suc), m.suc)
                    t(m.suc)
                }
            }

            t(k.suc)
            partial(tri_row_sum(sub_fn_2(f, g), k.suc), k.suc) = partial(tri_row_sum(f, k.suc), k.suc) - partial(tri_row_sum(g, k.suc), k.suc)
            triangular_sum(sub_fn_2(f, g), k.suc) = partial(tri_row_sum(sub_fn_2(f, g), k.suc), k.suc)
            triangular_sum(f, k.suc) = partial(tri_row_sum(f, k.suc), k.suc)
            triangular_sum(g, k.suc) = partial(tri_row_sum(g, k.suc), k.suc)
            triangular_sum(sub_fn_2(f, g), k.suc) = triangular_sum(f, k.suc) - triangular_sum(g, k.suc)
            p(k.suc)
        }
    }
    p(n)
}

/// Triangular sums of a bounded function are bounded.
theorem triangular_sum_upper_bound(f: (Nat, Nat) -> Real, bound: Real, n: Nat) {
    nonneg_fn_2(f) and is_upper_bound_fn_2(f, bound)
    implies
    triangular_sum(f, n) <= triangular_sum(const_fn_2(bound), n)
} by {
    if nonneg_fn_2(f) and is_upper_bound_fn_2(f, bound) {
        forall(i: Nat, j: Nat) {
            f(i, j) <= bound
            const_fn_2(bound, i, j) = bound
            f(i, j) <= const_fn_2(bound, i, j)
        }
        lte_fn_2(f, const_fn_2(bound))
        triangular_sum(f, n) <= triangular_sum(const_fn_2(bound), n)
    }
}

/// Triangular sum over increasing dimensions is monotone for nonnegative functions.
theorem triangular_sum_increasing(f: (Nat, Nat) -> Real, n: Nat) {
    nonneg_fn_2(f)
    implies
    triangular_sum(f, n) <= triangular_sum(f, n.suc)
} by {
    if nonneg_fn_2(f) {
        // Expand the definitions
        triangular_sum(f, n.suc) = partial(tri_row_sum(f, n.suc), n.suc)
        triangular_sum(f, n) = partial(tri_row_sum(f, n), n)

        // Split: triangular_sum(f, n.suc) = partial(tri_row_sum(f, n.suc), n) + tri_row_sum(f, n.suc, n)
        triangular_sum(f, n.suc) = partial(tri_row_sum(f, n.suc), n) + tri_row_sum(f, n.suc, n)

        // tri_row_sum(f, n.suc, n) = partial(f(n), 1) = f(n, 0) >= 0
        tri_row_sum(f, n.suc, n) = partial(f(n), n.suc - n)
        n.suc - n = Nat.1
        tri_row_sum(f, n.suc, n) = partial(f(n), Nat.1)
        partial(f(n), Nat.1) = partial(f(n), Nat.0) + f(n, Nat.0)
        partial(f(n), Nat.0) = Real.0
        tri_row_sum(f, n.suc, n) = f(n, Nat.0)
        f(n, Nat.0) >= Real.0
        tri_row_sum(f, n.suc, n) >= Real.0

        // Show: partial(tri_row_sum(f, n.suc), n) >= triangular_sum(f, n)
        // For i < n: tri_row_sum(f, n.suc, i) = tri_row_sum(f, n, i) + f(i, n-i) >= tri_row_sum(f, n, i)
        forall(i: Nat) {
            if i < n {
                i < n.suc
                i.suc <= n
                n >= i
                i <= n
                n - i + i = n
                (n - i + i).suc = (n - i).suc + i
                (n - i).suc + i = n.suc
                n.suc >= i
                n.suc - i = (n - i).suc
                tri_row_sum(f, n.suc, i) = partial(f(i), n.suc - i)
                tri_row_sum(f, n.suc, i) = partial(f(i), (n - i).suc)
                tri_row_sum(f, n, i) = partial(f(i), n - i)
                partial(f(i), (n - i).suc) = partial(f(i), n - i) + f(i, n - i)
                tri_row_sum(f, n.suc, i) = tri_row_sum(f, n, i) + f(i, n - i)
                f(i, n - i) >= Real.0
                tri_row_sum(f, n, i) + Real.0 <= tri_row_sum(f, n, i) + f(i, n - i)
                tri_row_sum(f, n.suc, i) >= tri_row_sum(f, n, i)
            }
        }

        // Now prove partial sums are monotone
        define r(k: Nat) -> Bool {
            k <= n
            implies
            partial(tri_row_sum(f, n.suc), k) >= partial(tri_row_sum(f, n), k)
        }

        r(Nat.0)

        forall(k: Nat) {
            if r(k) {
                if k.suc <= n {
                    k < n
                    tri_row_sum(f, n.suc, k) >= tri_row_sum(f, n, k)
                    partial(tri_row_sum(f, n.suc), k.suc) = partial(tri_row_sum(f, n.suc), k) + tri_row_sum(f, n.suc, k)
                    partial(tri_row_sum(f, n), k.suc) = partial(tri_row_sum(f, n), k) + tri_row_sum(f, n, k)
                    partial(tri_row_sum(f, n.suc), k) >= partial(tri_row_sum(f, n), k)
                    partial(tri_row_sum(f, n.suc), k) + tri_row_sum(f, n.suc, k) >= partial(tri_row_sum(f, n), k) + tri_row_sum(f, n, k)
                    partial(tri_row_sum(f, n.suc), k.suc) >= partial(tri_row_sum(f, n), k.suc)
                    r(k.suc)
                }
            }
        }

        forall(i: Nat) {
            if i < n {
                i < n.suc
                n >= i
                i <= n
                n - i + i = n
                (n - i + i).suc = (n - i).suc + i
                (n - i).suc + i = n.suc
                n.suc >= i
                n.suc - i = (n - i).suc
                tri_row_sum(f, n.suc, i) = partial(f(i), n.suc - i)
                tri_row_sum(f, n.suc, i) = partial(f(i), (n - i).suc)
                tri_row_sum(f, n, i) = partial(f(i), n - i)
                partial(f(i), (n - i).suc) = partial(f(i), n - i) + f(i, n - i)
                tri_row_sum(f, n.suc, i) = tri_row_sum(f, n, i) + f(i, n - i)
                diagonal(f, n, i) = f(i, n - i)
                tri_row_sum(f, n, i) + diagonal(f, n, i) = tri_row_sum(f, n, i) + f(i, n - i)
                tri_row_sum(f, n.suc, i) = tri_row_sum(f, n, i) + diagonal(f, n, i)
                add_fn(tri_row_sum(f, n), diagonal(f, n), i) = tri_row_sum(f, n, i) + diagonal(f, n, i)
                tri_row_sum(f, n.suc, i) = add_fn(tri_row_sum(f, n), diagonal(f, n), i)
            }
        }
        partial_pointwise_eq(tri_row_sum(f, n.suc), add_fn(tri_row_sum(f, n), diagonal(f, n)), n)
        partial(tri_row_sum(f, n.suc), n) = partial(add_fn(tri_row_sum(f, n), diagonal(f, n)), n)
        partial(add_fn(tri_row_sum(f, n), diagonal(f, n)), n) = partial(tri_row_sum(f, n), n) + partial(diagonal(f, n), n)
        partial(tri_row_sum(f, n.suc), n) = partial(tri_row_sum(f, n), n) + partial(diagonal(f, n), n)

        forall(i: Nat) {
            diagonal(f, n, i) = f(i, n - i)
            f(i, n - i) >= Real.0
            Real.0 <= f(i, n - i)
            Real.0 <= diagonal(f, n, i)
        }
        is_lower_bound(diagonal(f, n), Real.0)
        partial(diagonal(f, n), n) >= Real.0
        partial(tri_row_sum(f, n), n) + Real.0 <= partial(tri_row_sum(f, n), n) + partial(diagonal(f, n), n)
        partial(tri_row_sum(f, n), n) + partial(diagonal(f, n), n) = partial(tri_row_sum(f, n.suc), n)
        partial(tri_row_sum(f, n), n) + Real.0 <= partial(tri_row_sum(f, n.suc), n)
        partial(tri_row_sum(f, n), n) + Real.0 = partial(tri_row_sum(f, n), n)
        partial(tri_row_sum(f, n.suc), n) >= partial(tri_row_sum(f, n), n)
        n <= n implies partial(tri_row_sum(f, n.suc), n) >= partial(tri_row_sum(f, n), n)
        partial(tri_row_sum(f, n.suc), n) >= partial(tri_row_sum(f, n), n)
        triangular_sum(f, n) = partial(tri_row_sum(f, n), n)
        partial(tri_row_sum(f, n.suc), n) >= triangular_sum(f, n)
        triangular_sum(f, n.suc) = partial(tri_row_sum(f, n.suc), n) + tri_row_sum(f, n.suc, n)
        tri_row_sum(f, n.suc, n) >= Real.0
        partial(tri_row_sum(f, n.suc), n) + tri_row_sum(f, n.suc, n) >= triangular_sum(f, n) + Real.0
        triangular_sum(f, n.suc) >= triangular_sum(f, n)
    }
}

/// Triangular sums of a nonnegative function form an increasing sequence.
theorem triangular_sum_is_increasing(f: (Nat, Nat) -> Real) {
    nonneg_fn_2(f)
    implies
    is_increasing(triangular_sum_seq(f))
} by {
    if nonneg_fn_2(f) {
        forall(n: Nat) {
            triangular_sum_seq(f, n) = triangular_sum(f, n)
            triangular_sum_seq(f, n.suc) = triangular_sum(f, n.suc)
            triangular_sum(f, n) <= triangular_sum(f, n.suc)
            triangular_sum_seq(f, n) <= triangular_sum_seq(f, n.suc)
        }
    }
}

/// Triangular sums of a nonnegative function are monotone in the truncation bound.
theorem triangular_sum_is_monotone(f: (Nat, Nat) -> Real) {
    nonneg_fn_2(f)
    implies
    is_monotone(triangular_sum_seq(f))
} by {
    if nonneg_fn_2(f) {
        triangular_sum_is_increasing(f)
        is_increasing(triangular_sum_seq(f))
        increasing_is_monotone(triangular_sum_seq(f))
        is_monotone(triangular_sum_seq(f))
    }
}

//theorem triangular_sum_increasing(f: (Nat, Nat) -> Real, n: Nat) {
//    nonneg_fn_2(f)
//    implies
//    triangular_sum(f, n) <= triangular_sum(f, n.suc)
//} by {
//    if nonneg_fn_2(f) {
//        // Prove by induction on n
//        define q(m: Nat) -> Bool {
//            nonneg_fn_2(f)
//            implies
//            triangular_sum(f, m) <= triangular_sum(f, m.suc)
//        }
//
//        q(Nat.0)
//
//        forall(m: Nat) {
//            if q(m) {
//                if nonneg_fn_2(f) {
//                    triangular_sum(f, m.suc.suc) = partial(tri_row_sum(f, m.suc.suc), m.suc.suc)
//                    triangular_sum(f, m.suc.suc) = partial(tri_row_sum(f, m.suc.suc), m.suc) + tri_row_sum(f, m.suc.suc, m.suc)
//
//                    // Show tri_row_sum(f, m.suc.suc, m.suc) >= 0
//                    tri_row_sum(f, m.suc.suc, m.suc) = partial(f(m.suc), m.suc.suc - m.suc)
//                    tri_row_sum(f, m.suc.suc, m.suc) = partial(f(m.suc), Nat.1)
//                    partial(f(m.suc), Nat.1) = partial(f(m.suc), Nat.0) + f(m.suc, Nat.0)
//                    partial(f(m.suc), Nat.0) = Real.0
//                    tri_row_sum(f, m.suc.suc, m.suc) = f(m.suc, Nat.0)
//                    f(m.suc, Nat.0) >= Real.0
//                    tri_row_sum(f, m.suc.suc, m.suc) >= Real.0
//
//                    // Show partial(tri_row_sum(f, m.suc.suc), m.suc) >= triangular_sum(f, m.suc)
//                    // The key insight: for i <= m, tri_row_sum(f, m.suc.suc, i) includes one more element than tri_row_sum(f, m.suc, i)
//                    forall(i: Nat) {
//                        if i < m.suc {
//                            m.suc.suc - i = (m.suc - i).suc
//                            tri_row_sum(f, m.suc.suc, i) = partial(f(i), m.suc.suc - i)
//                            tri_row_sum(f, m.suc, i) = partial(f(i), m.suc - i)
//                            tri_row_sum(f, m.suc.suc, i) = partial(f(i), (m.suc - i).suc)
//                            partial(f(i), (m.suc - i).suc) = partial(f(i), m.suc - i) + f(i, m.suc - i)
//                            tri_row_sum(f, m.suc.suc, i) = tri_row_sum(f, m.suc, i) + f(i, m.suc - i)
//                            f(i, m.suc - i) >= Real.0
//                            tri_row_sum(f, m.suc.suc, i) >= tri_row_sum(f, m.suc, i)
//                        }
//                    }
//
//                    // Now show partial sums are monotone
//                    define r(k: Nat) -> Bool {
//                        k <= m.suc
//                        implies
//                        partial(tri_row_sum(f, m.suc.suc), k) >= partial(tri_row_sum(f, m.suc), k)
//                    }
//
//                    r(Nat.0)
//
//                    forall(k: Nat) {
//                        if r(k) {
//                            if k.suc <= m.suc {
//                                tri_row_sum(f, m.suc.suc, k) >= tri_row_sum(f, m.suc, k)
//                                partial(tri_row_sum(f, m.suc.suc), k.suc) = partial(tri_row_sum(f, m.suc.suc), k) + tri_row_sum(f, m.suc.suc, k)
//                                partial(tri_row_sum(f, m.suc), k.suc) = partial(tri_row_sum(f, m.suc), k) + tri_row_sum(f, m.suc, k)
//                                partial(tri_row_sum(f, m.suc.suc), k) >= partial(tri_row_sum(f, m.suc), k)
//                                partial(tri_row_sum(f, m.suc.suc), k.suc) >= partial(tri_row_sum(f, m.suc), k.suc)
//                                r(k.suc)
//                            }
//                        }
//                    }
//
//                    r(m.suc)
//                    partial(tri_row_sum(f, m.suc.suc), m.suc) >= partial(tri_row_sum(f, m.suc), m.suc)
//                    triangular_sum(f, m.suc) = partial(tri_row_sum(f, m.suc), m.suc)
//                    partial(tri_row_sum(f, m.suc.suc), m.suc) >= triangular_sum(f, m.suc)
//                    triangular_sum(f, m.suc.suc) >= triangular_sum(f, m.suc)
//                }
//                q(m.suc)
//            }
//        }
//
//        q(n)
//    }
//}

/// Recurrence relation for triangular sum of constant functions.
theorem triangular_sum_const_step(c: Real, n: Nat) {
    triangular_sum(const_fn_2(c), n.suc) = triangular_sum(const_fn_2(c), n) + c * nat_to_real(n.suc)
} by {
    // First prove helper: partial(const_fn_2(c, k), m) = c * nat_to_real(m)
    forall(k: Nat) {
        define p(m: Nat) -> Bool {
            partial(const_fn_2(c, k), m) = c * nat_to_real(m)
        }

        p(Nat.0)

        forall(m: Nat) {
            if p(m) {
                partial(const_fn_2(c, k), m.suc) = partial(const_fn_2(c, k), m) + const_fn_2(c, k, m)
                const_fn_2(c, k, m) = c
                partial(const_fn_2(c, k), m.suc) = c * nat_to_real(m) + c
                nat_to_real(m.suc) = nat_to_real(m) + Real.1
                c * nat_to_real(m.suc) = c * (nat_to_real(m) + Real.1)
                c * nat_to_real(m.suc) = c * nat_to_real(m) + c
                partial(const_fn_2(c, k), m.suc) = c * nat_to_real(m.suc)
                p(m.suc)
            }
        }
    }

    // From helper: tri_row_sum(const_fn_2(c), n, i) = c * nat_to_real(n - i)

    triangular_sum(const_fn_2(c), n.suc) = partial(tri_row_sum(const_fn_2(c), n.suc), n.suc)
    triangular_sum(const_fn_2(c), n) = partial(tri_row_sum(const_fn_2(c), n), n)

    // Show that partial(tri_row_sum(const_fn_2(c), n.suc), n.suc)
    //            = partial(tri_row_sum(const_fn_2(c), n.suc), n) + tri_row_sum(const_fn_2(c), n.suc, n)
    triangular_sum(const_fn_2(c), n.suc) = partial(tri_row_sum(const_fn_2(c), n.suc), n) + tri_row_sum(const_fn_2(c), n.suc, n)

    // tri_row_sum(const_fn_2(c), n.suc, n) = partial(const_fn_2(c, n), n.suc - n) = partial(const_fn_2(c, n), 1) = c
    tri_row_sum(const_fn_2(c), n.suc, n) = partial(const_fn_2(c, n), n.suc - n)
    n.suc - n = Nat.1
    tri_row_sum(const_fn_2(c), n.suc, n) = partial(const_fn_2(c, n), Nat.1)
    partial(const_fn_2(c, n), Nat.1) = c * nat_to_real(Nat.1)
    nat_to_real(Nat.1) = nat_to_real(Nat.0) + Real.1
    nat_to_real(Nat.0) = Real.0
    nat_to_real(Nat.1) = Real.1
    tri_row_sum(const_fn_2(c), n.suc, n) = c

    // Show that partial(tri_row_sum(const_fn_2(c), n.suc), n) = triangular_sum(const_fn_2(c), n)
    // This requires showing tri_row_sum(const_fn_2(c), n.suc, i) = tri_row_sum(const_fn_2(c), n, i) for i < n
    define q(i: Nat) -> Bool {
        i < n.suc
        implies
        tri_row_sum(const_fn_2(c), n.suc, i) = partial(const_fn_2(c, i), n.suc - i)
    }

    forall(i: Nat) {
        if i < n.suc {
            tri_row_sum(const_fn_2(c), n.suc, i) = partial(const_fn_2(c, i), n.suc - i)
        }
    }

    // For i < n: n.suc - i = (n - i).suc
    define r(i: Nat) -> Bool {
        i < n
        implies
        tri_row_sum(const_fn_2(c), n.suc, i) = tri_row_sum(const_fn_2(c), n, i) + c
    }

        forall(i: Nat) {
            if i < n {
            i < n.suc
            i.suc <= n
            i <= n
            n >= i
            n.suc >= i
            n - i + i = n
            (n - i + i).suc = (n - i).suc + i
            (n - i + i).suc = n.suc
            (n - i).suc + i = n.suc
            n.suc - i = (n - i).suc
            tri_row_sum(const_fn_2(c), n.suc, i) = partial(const_fn_2(c, i), n.suc - i)
            tri_row_sum(const_fn_2(c), n.suc, i) = partial(const_fn_2(c, i), (n - i).suc)
            tri_row_sum(const_fn_2(c), n, i) = partial(const_fn_2(c, i), n - i)

            // Reprove that partial(const_fn_2(c, i), m) = c * nat_to_real(m) for m = (n-i).suc
            define helper_suc(m: Nat) -> Bool {
                partial(const_fn_2(c, i), m) = c * nat_to_real(m)
            }

            helper_suc(Nat.0)

            forall(m: Nat) {
                if helper_suc(m) {
                    partial(const_fn_2(c, i), m) = c * nat_to_real(m)
                    partial(const_fn_2(c, i), m.suc) = partial(const_fn_2(c, i), m) + const_fn_2(c, i, m)
                    const_fn_2(c, i, m) = c
                    nat_to_real(m.suc) = nat_to_real(m) + Real.1
                    partial(const_fn_2(c, i), m.suc) = c * nat_to_real(m) + c
                    c * nat_to_real(m.suc) = c * (nat_to_real(m) + Real.1)
                    c * nat_to_real(m.suc) = c * nat_to_real(m) + c
                    partial(const_fn_2(c, i), m.suc) = c * nat_to_real(m.suc)
                    helper_suc(m.suc)
                }
            }

            helper_suc((n - i).suc)
            helper_suc(n - i)
            partial(const_fn_2(c, i), (n - i).suc) = c * nat_to_real((n - i).suc)
            partial(const_fn_2(c, i), n - i) = c * nat_to_real(n - i)

            tri_row_sum(const_fn_2(c), n.suc, i) = c * nat_to_real((n - i).suc)
            tri_row_sum(const_fn_2(c), n, i) = c * nat_to_real(n - i)
            nat_to_real((n - i).suc) = nat_to_real(n - i) + Real.1
            c * nat_to_real((n - i).suc) = c * nat_to_real(n - i) + c
            tri_row_sum(const_fn_2(c), n.suc, i) = tri_row_sum(const_fn_2(c), n, i) + c
            r(i)
        }
    }

    // Now compute partial(tri_row_sum(const_fn_2(c), n.suc), n)
    define s(k: Nat) -> Bool {
        k <= n
        implies
        partial(tri_row_sum(const_fn_2(c), n.suc), k) = partial(tri_row_sum(const_fn_2(c), n), k) + c * nat_to_real(k)
    }

    partial(tri_row_sum(const_fn_2(c), n.suc), Nat.0) = Real.0
    partial(tri_row_sum(const_fn_2(c), n), Nat.0) = Real.0
    nat_to_real(Nat.0) = Real.0
    c * nat_to_real(Nat.0) = Real.0
    partial(tri_row_sum(const_fn_2(c), n), Nat.0) + c * nat_to_real(Nat.0) = Real.0
    s(Nat.0)

    forall(k: Nat) {
        if s(k) {
            if k.suc <= n {
                k < n
                tri_row_sum(const_fn_2(c), n.suc, k) = tri_row_sum(const_fn_2(c), n, k) + c
                partial(tri_row_sum(const_fn_2(c), n.suc), k.suc) = partial(tri_row_sum(const_fn_2(c), n.suc), k) + tri_row_sum(const_fn_2(c), n.suc, k)
                partial(tri_row_sum(const_fn_2(c), n), k.suc) = partial(tri_row_sum(const_fn_2(c), n), k) + tri_row_sum(const_fn_2(c), n, k)
                partial(tri_row_sum(const_fn_2(c), n.suc), k) = partial(tri_row_sum(const_fn_2(c), n), k) + c * nat_to_real(k)
                partial(tri_row_sum(const_fn_2(c), n.suc), k.suc) = partial(tri_row_sum(const_fn_2(c), n), k) + c * nat_to_real(k) + tri_row_sum(const_fn_2(c), n, k) + c
                c * nat_to_real(k) + (partial(tri_row_sum(const_fn_2(c), n), k) + tri_row_sum(const_fn_2(c), n, k)) = c * nat_to_real(k) + partial(tri_row_sum(const_fn_2(c), n), k) + tri_row_sum(const_fn_2(c), n, k)
                c * nat_to_real(k) + partial(tri_row_sum(const_fn_2(c), n), k) = partial(tri_row_sum(const_fn_2(c), n), k) + c * nat_to_real(k)
                c * nat_to_real(k) + partial(tri_row_sum(const_fn_2(c), n), k.suc) = partial(tri_row_sum(const_fn_2(c), n), k.suc) + c * nat_to_real(k)
                partial(tri_row_sum(const_fn_2(c), n.suc), k.suc) = partial(tri_row_sum(const_fn_2(c), n), k.suc) + c * nat_to_real(k) + c
                nat_to_real(k.suc) = nat_to_real(k) + Real.1
                c * nat_to_real(k.suc) = c * nat_to_real(k) + c
                partial(tri_row_sum(const_fn_2(c), n.suc), k.suc) = partial(tri_row_sum(const_fn_2(c), n), k.suc) + c * nat_to_real(k.suc)
                s(k.suc)
            }
        }
    }
    forall(i: Nat) {
        if i < n {
            r(i)
            tri_row_sum(const_fn_2(c), n.suc, i) = tri_row_sum(const_fn_2(c), n, i) + c
            const_seq(c, i) = c
            add_fn(tri_row_sum(const_fn_2(c), n), const_seq(c), i) = tri_row_sum(const_fn_2(c), n, i) + const_seq(c, i)
            add_fn(tri_row_sum(const_fn_2(c), n), const_seq(c), i) = tri_row_sum(const_fn_2(c), n, i) + c
            tri_row_sum(const_fn_2(c), n, i) + c = add_fn(tri_row_sum(const_fn_2(c), n), const_seq(c), i)
                tri_row_sum(const_fn_2(c), n.suc, i) = add_fn(tri_row_sum(const_fn_2(c), n), const_seq(c), i)
            }
        }
    partial_pointwise_eq(tri_row_sum(const_fn_2(c), n.suc), add_fn(tri_row_sum(const_fn_2(c), n), const_seq(c)), n)
    partial(tri_row_sum(const_fn_2(c), n.suc), n) = partial(add_fn(tri_row_sum(const_fn_2(c), n), const_seq(c)), n)
    partial(add_fn(tri_row_sum(const_fn_2(c), n), const_seq(c)), n) = partial(tri_row_sum(const_fn_2(c), n), n) + partial(const_seq(c), n)

    define const_partial(m: Nat) -> Bool {
        partial(const_seq(c), m) = c * nat_to_real(m)
    }

    partial(const_seq(c), Nat.0) = Real.0
    nat_to_real(Nat.0) = Real.0
    c * nat_to_real(Nat.0) = Real.0
    const_partial(Nat.0)

    forall(m: Nat) {
        if const_partial(m) {
            partial(const_seq(c), m) = c * nat_to_real(m)
            partial(const_seq(c), m.suc) = partial(const_seq(c), m) + const_seq(c, m)
            partial(const_seq(c), m.suc) = c * nat_to_real(m) + const_seq(c, m)
            const_seq(c, m) = c
            partial(const_seq(c), m.suc) = c * nat_to_real(m) + c
            nat_to_real(m.suc) = nat_to_real(m) + Real.1
            c * nat_to_real(m.suc) = c * (nat_to_real(m) + Real.1)
            c * nat_to_real(m.suc) = c * nat_to_real(m) + c
            partial(const_seq(c), m.suc) = c * nat_to_real(m.suc)
            const_partial(m.suc)
        }
    }

    const_partial(n)
    partial(const_seq(c), n) = c * nat_to_real(n)
    partial(tri_row_sum(const_fn_2(c), n.suc), n) = partial(tri_row_sum(const_fn_2(c), n), n) + partial(const_seq(c), n)
    partial(tri_row_sum(const_fn_2(c), n.suc), n) = partial(tri_row_sum(const_fn_2(c), n), n) + c * nat_to_real(n)
    n <= n implies partial(tri_row_sum(const_fn_2(c), n.suc), n) = partial(tri_row_sum(const_fn_2(c), n), n) + c * nat_to_real(n)
    s(n)
    partial(tri_row_sum(const_fn_2(c), n.suc), n) = partial(tri_row_sum(const_fn_2(c), n), n) + c * nat_to_real(n)
    triangular_sum(const_fn_2(c), n) = partial(tri_row_sum(const_fn_2(c), n), n)
    partial(tri_row_sum(const_fn_2(c), n.suc), n) = triangular_sum(const_fn_2(c), n) + c * nat_to_real(n)

    triangular_sum(const_fn_2(c), n.suc) = partial(tri_row_sum(const_fn_2(c), n.suc), n) + c
    triangular_sum(const_fn_2(c), n.suc) = triangular_sum(const_fn_2(c), n) + c * nat_to_real(n) + c
    nat_to_real(n.suc) = nat_to_real(n) + Real.1
    c * nat_to_real(n.suc) = c * nat_to_real(n) + c
    triangular_sum(const_fn_2(c), n.suc) = triangular_sum(const_fn_2(c), n) + c * nat_to_real(n.suc)
}

/// Triangular sum of a constant function equals c * n * (n+1) / 2.
theorem triangular_sum_const(c: Real, n: Nat) {
    triangular_sum(const_fn_2(c), n) = c * nat_to_real(n) * nat_to_real(n.suc) * Real.one_half
} by {
    define p(k: Nat) -> Bool {
        triangular_sum(const_fn_2(c), k) = c * nat_to_real(k) * nat_to_real(k.suc) * Real.one_half
    }

    triangular_sum(const_fn_2(c), Nat.0) = Real.0
    nat_to_real(Nat.0) = Real.0
    c * nat_to_real(Nat.0) = Real.0
    c * nat_to_real(Nat.0) * nat_to_real(Nat.0.suc) = Real.0
    c * nat_to_real(Nat.0) * nat_to_real(Nat.0.suc) * Real.one_half = Real.0
    p(Nat.0)

    forall(k: Nat) {
        if p(k) {
            triangular_sum(const_fn_2(c), k) = c * nat_to_real(k) * nat_to_real(k.suc) * Real.one_half
            triangular_sum(const_fn_2(c), k.suc) = triangular_sum(const_fn_2(c), k) + c * nat_to_real(k.suc)
            triangular_sum(const_fn_2(c), k.suc) = c * nat_to_real(k) * nat_to_real(k.suc) * Real.one_half + c * nat_to_real(k.suc)

            // Simplify: c * k * (k+1) / 2 + c * (k+1) = c * (k+1) * [k/2 + 1] = c * (k+1) * (k+2) / 2
            c * nat_to_real(k.suc) * Real.1 + c * nat_to_real(k.suc) * (nat_to_real(k) * Real.one_half) = c * nat_to_real(k.suc) * (Real.1 + nat_to_real(k) * Real.one_half)
            c * nat_to_real(k.suc) * (nat_to_real(k) * Real.one_half) = c * nat_to_real(k.suc) * nat_to_real(k) * Real.one_half
            nat_to_real(k.suc) * (c * nat_to_real(k)) = nat_to_real(k.suc) * c * nat_to_real(k)
            c * nat_to_real(k) * nat_to_real(k.suc) * Real.one_half = Real.one_half * (c * nat_to_real(k) * nat_to_real(k.suc))
            nat_to_real(k.suc) * (c * nat_to_real(k)) = c * nat_to_real(k) * nat_to_real(k.suc)
            c * nat_to_real(k.suc) = nat_to_real(k.suc) * c
            nat_to_real(k) * Real.one_half = Real.one_half * nat_to_real(k)
            Real.one_half * nat_to_real(k) + Real.1 = Real.1 + Real.one_half * nat_to_real(k)
            c * nat_to_real(k.suc) + Real.one_half * (c * nat_to_real(k) * nat_to_real(k.suc)) = Real.one_half * (c * nat_to_real(k) * nat_to_real(k.suc)) + c * nat_to_real(k.suc)
            nat_to_real(k.suc) * c * Real.1 = nat_to_real(k.suc) * c
            c * nat_to_real(k) * nat_to_real(k.suc) * Real.one_half + c * nat_to_real(k.suc) = c * nat_to_real(k.suc) * (nat_to_real(k) * Real.one_half + Real.1)
            nat_to_real(k) * Real.one_half + Real.1 = (nat_to_real(k) + Real.1 + Real.1) * Real.one_half
            Real.1 + Real.1 = Real.one_half + Real.one_half + Real.one_half + Real.one_half
            nat_to_real(k) + Real.1 + Real.1 = nat_to_real(k) + (Real.one_half + Real.one_half) + (Real.one_half + Real.one_half)
            nat_to_real(k.suc.suc) = nat_to_real(k.suc) + Real.1
            nat_to_real(k.suc) = nat_to_real(k) + Real.1
            nat_to_real(k.suc.suc) = nat_to_real(k) + Real.1 + Real.1
            (nat_to_real(k) + Real.1 + Real.1) * Real.one_half = nat_to_real(k.suc.suc) * Real.one_half
            nat_to_real(k) * Real.one_half + Real.1 = nat_to_real(k.suc.suc) * Real.one_half
            c * nat_to_real(k.suc) * (nat_to_real(k) * Real.one_half + Real.1) = c * nat_to_real(k.suc) * nat_to_real(k.suc.suc) * Real.one_half
            triangular_sum(const_fn_2(c), k.suc) = c * nat_to_real(k.suc) * nat_to_real(k.suc.suc) * Real.one_half
            p(k.suc)
        }
    }
    p(n)
}

/// Triangular sum is bounded by rectangular sum.
theorem triangular_sum_le_rectangular(f: (Nat, Nat) -> Real, n: Nat) {
    nonneg_fn_2(f)
    implies
    triangular_sum(f, n) <= rectangular_sum(f, n, n)
} by {
    if nonneg_fn_2(f) {
        // For each i < n, show tri_row_sum(f, n, i) <= row_sum(f, n, i)
        forall(i: Nat) {
            if i < n {
                // tri_row_sum(f, n, i) = partial(f(i), n - i)
                // row_sum(f, n, i) = partial(f(i), n)
                // Since n - i <= n and f is nonnegative, partial(f(i), n-i) <= partial(f(i), n)

                tri_row_sum(f, n, i) = partial(f(i), n - i)
                row_sum(f, n, i) = partial(f(i), n)

                // Prove by induction on distance from n - i to n
                i < n
                i.suc <= n
                n >= i

                // Induct on m, the extra distance: partial(f(i), n - i + m) is increasing in m
                define p(m: Nat) -> Bool {
                    n - i + m <= n
                    implies
                    partial(f(i), n - i) <= partial(f(i), n - i + m)
                }

                p(Nat.0)

                forall(m: Nat) {
                    if p(m) {
                        if n - i + m.suc <= n {
                            n - i + m < n
                            n - i + m.suc <= n
                            partial(f(i), n - i) <= partial(f(i), n - i + m)
                            f(i, n - i + m) >= Real.0
                            partial(f(i), (n - i + m).suc) = partial(f(i), n - i + m) + f(i, n - i + m)
                            partial(f(i), n - i) + Real.0 <= partial(f(i), n - i + m) + f(i, n - i + m)
                            partial(f(i), n - i) <= partial(f(i), (n - i + m).suc)
                            n - i + m.suc = (n - i + m).suc
                            partial(f(i), n - i) <= partial(f(i), n - i + m.suc)
                            p(m.suc)
                        }
                    }
                }

                // Apply with m such that n - i + m = n, i.e., m = i
                n - i + i = n
                p(i)
                partial(f(i), n - i) <= partial(f(i), n)
                row_sum(f, n, i) >= tri_row_sum(f, n, i)
                tri_row_sum(f, n, i) <= row_sum(f, n, i)
            }
        }

        // Now sum over i to get triangular_sum(f, n) <= rectangular_sum(f, n, n)
        define q(k: Nat) -> Bool {
            k <= n
            implies
            partial(tri_row_sum(f, n), k) <= partial(row_sum(f, n), k)
        }

        q(Nat.0)

        forall(k: Nat) {
            if q(k) {
                if k.suc <= n {
                    k < n
                    tri_row_sum(f, n, k) <= row_sum(f, n, k)
                    partial(tri_row_sum(f, n), k) <= partial(row_sum(f, n), k)
                    partial(tri_row_sum(f, n), k.suc) = partial(tri_row_sum(f, n), k) + tri_row_sum(f, n, k)
                    partial(row_sum(f, n), k.suc) = partial(row_sum(f, n), k) + row_sum(f, n, k)
                    partial(tri_row_sum(f, n), k) + tri_row_sum(f, n, k) <= partial(row_sum(f, n), k) + row_sum(f, n, k)
                    partial(tri_row_sum(f, n), k.suc) <= partial(row_sum(f, n), k.suc)
                    q(k.suc)
                }
            }
        }

        q(n)
        partial(tri_row_sum(f, n), n) <= partial(row_sum(f, n), n)
        triangular_sum(f, n) = partial(tri_row_sum(f, n), n)
        rectangular_sum(f, n, n) = partial(row_sum(f, n), n)
        triangular_sum(f, n) <= rectangular_sum(f, n, n)
    }
}

/// Diagonal sum of flip(f) equals diagonal sum of f.
theorem diagonal_sum_flip(f: (Nat, Nat) -> Real, n: Nat) {
    diagonal_sum(flip(f), n) = diagonal_sum(f, n)
} by {
    // diagonal_sum is defined as partial(diagonal(f, n), n.suc)
    diagonal_sum(flip(f), n) = partial(diagonal(flip(f), n), n.suc)
    diagonal_sum(f, n) = partial(diagonal(f, n), n.suc)

    // Show that diagonal(flip(f), n, i) = diagonal(f, n, n - i)
    forall(i: Nat) {
        if i <= n {
            diagonal(flip(f), n, i) = flip(f, i, n - i)
            flip(f, i, n - i) = f(n - i, i)
            diagonal(f, n, n - i) = f(n - i, n - (n - i))
            n - (n - i) = i
            diagonal(f, n, n - i) = f(n - i, i)
            diagonal(flip(f), n, i) = diagonal(f, n, n - i)
        }
    }

    // Show that diagonal(flip(f), n) and reverse_index(diagonal(f, n), n) agree on all indices
    forall(i: Nat) {
        if i < n.suc {
            i <= n
            diagonal(flip(f), n, i) = diagonal(f, n, n - i)
            reverse_index(diagonal(f, n), n, i) = diagonal(f, n, n - i)
            diagonal(flip(f), n, i) = reverse_index(diagonal(f, n), n, i)
        }
    }

    // Use partial_pointwise_eq to show partial sums are equal
    partial_pointwise_eq(diagonal(flip(f), n), reverse_index(diagonal(f, n), n), n.suc)
    partial(diagonal(flip(f), n), n.suc) = partial(reverse_index(diagonal(f, n), n), n.suc)

    // Use partial_reverse to relate reverse_index back to diagonal(f, n)
    partial(reverse_index(diagonal(f, n), n), n.suc) = partial(diagonal(f, n), n.suc)

    diagonal_sum(flip(f), n) = diagonal_sum(f, n)
}

/// Triangular sum of flip(f) equals triangular sum of f.
theorem triangular_sum_flip(f: (Nat, Nat) -> Real, n: Nat) {
    triangular_sum(flip(f), n) = triangular_sum(f, n)
} by {
    define p(m: Nat) -> Bool {
        triangular_sum(flip(f), m) = triangular_sum(f, m)
    }

    p(Nat.0)

    forall(m: Nat) {
        if p(m) {
            // Use the recurrence relation
            triangular_sum(flip(f), m.suc) = triangular_sum(flip(f), m) + diagonal_sum(flip(f), m)
            triangular_sum(f, m.suc) = triangular_sum(f, m) + diagonal_sum(f, m)

            // By IH
            triangular_sum(flip(f), m) = triangular_sum(f, m)

            // By diagonal_sum_flip
            diagonal_sum(flip(f), m) = diagonal_sum(f, m)

            // Therefore
            triangular_sum(flip(f), m.suc) = triangular_sum(f, m) + diagonal_sum(f, m)
            triangular_sum(flip(f), m.suc) = triangular_sum(f, m.suc)

            p(m.suc)
        }
    }
    p(n)
}

/// Helper: relate shifted partial sums.
define shift_fn(g: Nat -> Real, offset: Nat, j: Nat) -> Real {
    g(offset + j)
}

/// Split a partial sum at a boundary.
theorem partial_split(g: Nat -> Real, n: Nat, k: Nat) {
    partial(g, n + k) = partial(g, n) + partial(shift_fn(g, n), k)
} by {
    define p(m: Nat) -> Bool {
        partial(g, n + m) = partial(g, n) + partial(shift_fn(g, n), m)
    }

    p(Nat.0)

    forall(m: Nat) {
        if p(m) {
            partial(g, n + m.suc) = partial(g, n + m) + g(n + m)
            partial(g, n + m) = partial(g, n) + partial(shift_fn(g, n), m)
            partial(g, n + m.suc) = partial(g, n) + partial(shift_fn(g, n), m) + g(n + m)

            shift_fn(g, n, m) = g(n + m)
            partial(shift_fn(g, n), m.suc) = partial(shift_fn(g, n), m) + shift_fn(g, n, m)
            partial(shift_fn(g, n), m.suc) = partial(shift_fn(g, n), m) + g(n + m)

            partial(g, n + m.suc) = partial(g, n) + partial(shift_fn(g, n), m.suc)
            p(m.suc)
        }
    }
    p(k)
}

/// Triangle decomposition: a triangular sum equals a rectangular sum plus two shifted triangular sums.
/// The triangular region {(i,j) : i+j < m+n} decomposes into:
/// 1. Rectangle {(i,j) : i < m, j < n}
/// 2. Shifted triangle {(i,j) : i+j < m} shifted by n columns
/// 3. Shifted triangle {(i,j) : i+j < n} shifted by m rows
theorem triangular_sum_decomposition(f: (Nat, Nat) -> Real, m: Nat, n: Nat) {
    triangular_sum(f, m + n) =
        rectangular_sum(f, m, n) +
        triangular_sum(shift_cols(f, n), m) +
        triangular_sum(shift_rows(f, m), n)
} by {
    // Step 1: Show that for i < m, the row sum splits correctly
    forall(i: Nat) {
        if i < m {
            i < m
            i.suc <= m
            m >= i

            // Prove m + n - i = n + (m - i) explicitly
            // Since m >= i, we have m = i + (m-i), so m + n = i + (m-i) + n = i + n + (m-i)
            // Thus m + n - i = n + (m-i)
            (m - i) + i = m
            (m - i) + i + n = m + n
            n + (m - i) + i = m + n
            m + n - i = n + (m - i)

            tri_row_sum(f, m + n, i) = partial(f(i), m + n - i)
            tri_row_sum(f, m + n, i) = partial(f(i), n + (m - i))

            // Use partial_split to decompose this sum            partial(f(i), n + (m - i)) = partial(f(i), n) + partial(shift_fn(f(i), n), m - i)

            // Show shift_fn(f(i), n) equals shift_cols(f, n, i)
            forall(j: Nat) {
                shift_fn(f(i), n, j) = f(i, n + j)
                shift_cols(f, n, i, j) = f(i, n + j)
                shift_fn(f(i), n, j) = shift_cols(f, n, i, j)
            }

            partial(shift_fn(f(i), n), m - i) = partial(shift_cols(f, n, i), m - i)

            tri_row_sum(f, m + n, i) = partial(f(i), n) + partial(shift_cols(f, n, i), m - i)

            row_sum(f, n, i) = partial(f(i), n)
            tri_row_sum(shift_cols(f, n), m, i) = partial(shift_cols(f, n, i), m - i)

            tri_row_sum(f, m + n, i) = row_sum(f, n, i) + tri_row_sum(shift_cols(f, n), m, i)
        }
    }

    // Step 2: Show that for i >= m and i < m+n, the row sum equals shift_rows contribution
    forall(i: Nat) {
        if i >= m and i < m + n {
            // When i >= m, write i = m + i' where i' = i - m
            // Then tri_row_sum(f, m+n, i) = partial(f(i), m+n-i) = partial(f(m+i'), n-i')
            i >= m
            m + n > i
            m + n - i > Nat.0

            tri_row_sum(f, m + n, i) = partial(f(i), m + n - i)

            // Show that i - m < n and m + n - i = n - (i - m)
            // From i >= m, we have (i-m) + m = i
            // From i < m+n, we have (i-m) + m < m+n
            (i - m) + m = i
            (i - m) + m < m + n

            // Therefore i-m < n
            // Also, m + n - i = m + n - ((i-m) + m) = n - (i-m)
            // This follows from nat arithmetic

            tri_row_sum(shift_rows(f, m), n, i - m) = partial(shift_rows(f, m, i - m), n - (i - m))
            let i_minus_m: Nat satisfy {
                m + i_minus_m = i
            }
            let m_plus_n_minus_i: Nat satisfy {
                i + m_plus_n_minus_i = m + n
            }
            i - m = i_minus_m
            m + n - i = m_plus_n_minus_i
            m_plus_n_minus_i + i_minus_m = n
            m + n - i = n - (i - m)
            tri_row_sum(f, m + n, i) = partial(f(i), n - (i - m))

            // Now f(i) = f(m + (i-m)) = shift_rows(f, m, i-m)
            forall(j: Nat) {
                f(i, j) = f(m + (i - m), j)
                shift_rows(f, m, i - m, j) = f(m + (i - m), j)
                f(i, j) = shift_rows(f, m, i - m, j)
            }

            partial(f(i), n - (i - m)) = partial(shift_rows(f, m, i - m), n - (i - m))

            tri_row_sum(shift_rows(f, m), n, i - m) = partial(shift_rows(f, m, i - m), n - (i - m))

            tri_row_sum(f, m + n, i) = tri_row_sum(shift_rows(f, m), n, i - m)
        }
    }

    // Step 3: Combine the row sums
    triangular_sum(f, m + n) = partial(tri_row_sum(f, m + n), m + n)

    // Use partial_split to split at position m
    partial(tri_row_sum(f, m + n), m + n) = partial(tri_row_sum(f, m + n), m) + partial(shift_fn(tri_row_sum(f, m + n), m), n)

    // For i < m, we showed tri_row_sum(f, m+n, i) = row_sum(f, n, i) + tri_row_sum(shift_cols(f, n), m, i)
    forall(i: Nat) {
        if i < m {
            tri_row_sum(f, m + n, i) = row_sum(f, n, i) + tri_row_sum(shift_cols(f, n), m, i)
            add_fn(row_sum(f, n), tri_row_sum(shift_cols(f, n), m), i) = row_sum(f, n, i) + tri_row_sum(shift_cols(f, n), m, i)
            tri_row_sum(f, m + n, i) = add_fn(row_sum(f, n), tri_row_sum(shift_cols(f, n), m), i)
        }
    }

    partial_pointwise_eq(tri_row_sum(f, m + n), add_fn(row_sum(f, n), tri_row_sum(shift_cols(f, n), m)), m)
    partial(tri_row_sum(f, m + n), m) = partial(add_fn(row_sum(f, n), tri_row_sum(shift_cols(f, n), m)), m)
    partial(add_fn(row_sum(f, n), tri_row_sum(shift_cols(f, n), m)), m) = partial(row_sum(f, n), m) + partial(tri_row_sum(shift_cols(f, n), m), m)
    partial(row_sum(f, n), m) = rectangular_sum(f, m, n)
    partial(tri_row_sum(shift_cols(f, n), m), m) = triangular_sum(shift_cols(f, n), m)
    partial(tri_row_sum(f, m + n), m) = rectangular_sum(f, m, n) + triangular_sum(shift_cols(f, n), m)

    // For i >= m, we showed tri_row_sum(f, m+n, i) = tri_row_sum(shift_rows(f, m), n, i-m)
    // So shift_fn(tri_row_sum(f, m+n), m, j) = tri_row_sum(f, m+n, m+j) = tri_row_sum(shift_rows(f, m), n, j)
    forall(j: Nat) {
        if j < n {
            m + j < m + n
            m + j >= m
            tri_row_sum(f, m + n, m + j) = tri_row_sum(shift_rows(f, m), n, (m + j) - m)
            (m + j) - m = j
            tri_row_sum(f, m + n, m + j) = tri_row_sum(shift_rows(f, m), n, j)
            shift_fn(tri_row_sum(f, m + n), m, j) = tri_row_sum(f, m + n, m + j)
            shift_fn(tri_row_sum(f, m + n), m, j) = tri_row_sum(shift_rows(f, m), n, j)
        }
    }

    partial_pointwise_eq(shift_fn(tri_row_sum(f, m + n), m), tri_row_sum(shift_rows(f, m), n), n)
    partial(shift_fn(tri_row_sum(f, m + n), m), n) = partial(tri_row_sum(shift_rows(f, m), n), n)
    partial(tri_row_sum(shift_rows(f, m), n), n) = triangular_sum(shift_rows(f, m), n)

    triangular_sum(f, m + n) = partial(tri_row_sum(f, m + n), m) + partial(shift_fn(tri_row_sum(f, m + n), m), n)
    triangular_sum(f, m + n) = rectangular_sum(f, m, n) + triangular_sum(shift_cols(f, n), m) + triangular_sum(shift_rows(f, m), n)
}

/// Rectangular sum is bounded by triangular sum.
theorem rectangular_sum_le_triangular(f: (Nat, Nat) -> Real, m: Nat, n: Nat) {
    nonneg_fn_2(f)
    implies
    rectangular_sum(f, m, n) <= triangular_sum(f, m + n)
} by {
    if nonneg_fn_2(f) {
        // Use the decomposition theorem
        triangular_sum(f, m + n) = rectangular_sum(f, m, n) + triangular_sum(shift_cols(f, n), m) + triangular_sum(shift_rows(f, m), n)

        // Show that shifted functions are also nonnegative
        forall(i: Nat, j: Nat) {
            shift_cols(f, n, i, j) = f(i, n + j)
            f(i, n + j) >= Real.0
            shift_cols(f, n, i, j) >= Real.0
        }
        nonneg_fn_2(shift_cols(f, n))

        forall(i: Nat, j: Nat) {
            shift_rows(f, m, i, j) = f(m + i, j)
            f(m + i, j) >= Real.0
            shift_rows(f, m, i, j) >= Real.0
        }
        nonneg_fn_2(shift_rows(f, m))

        // Therefore the shifted triangular sums are nonnegative
        triangular_sum(shift_cols(f, n), m) >= Real.0
        triangular_sum(shift_rows(f, m), n) >= Real.0

        // Since both shifted sums are nonnegative, adding them to rectangular_sum doesn't decrease it
        rectangular_sum(f, m, n) + Real.0 <= rectangular_sum(f, m, n) + triangular_sum(shift_cols(f, n), m)
        rectangular_sum(f, m, n) <= rectangular_sum(f, m, n) + triangular_sum(shift_cols(f, n), m)
        rectangular_sum(f, m, n) + triangular_sum(shift_cols(f, n), m) + Real.0 <= rectangular_sum(f, m, n) + triangular_sum(shift_cols(f, n), m) + triangular_sum(shift_rows(f, m), n)
        rectangular_sum(f, m, n) <= rectangular_sum(f, m, n) + triangular_sum(shift_cols(f, n), m) + triangular_sum(shift_rows(f, m), n)

        // From decomposition, the right side equals triangular_sum(f, m + n)
        rectangular_sum(f, m, n) <= triangular_sum(f, m + n)
    }
}

/// For nonnegative functions, triangular sum at 2n is greater than or equal to square sum at n.
theorem triangular_sum_ge_square_sum(f: (Nat, Nat) -> Real, n: Nat) {
    nonneg_fn_2(f)
    implies
    square_sum(f, n) <= triangular_sum(f, n + n)
} by {
    if nonneg_fn_2(f) {
        square_sum(f, n) = rectangular_sum(f, n, n)
        rectangular_sum(f, n, n) <= triangular_sum(f, n + n)
        square_sum(f, n) <= triangular_sum(f, n + n)
    }
}

/// For nonnegative functions, triangular sum is bounded by square sum.
theorem triangular_sum_le_square_sum(f: (Nat, Nat) -> Real, n: Nat) {
    nonneg_fn_2(f)
    implies
    triangular_sum(f, n) <= square_sum(f, n)
} by {
    if nonneg_fn_2(f) {
        square_sum(f, n) = rectangular_sum(f, n, n)
        triangular_sum(f, n) <= rectangular_sum(f, n, n)
        triangular_sum(f, n) <= square_sum(f, n)
    }
}

/// For nonnegative functions with convergent double sum, the triangular sum also converges to the double sum.
theorem triangular_sum_converges_to_double_sum(f: (Nat, Nat) -> Real) {
    nonneg_fn_2(f) and double_sum_converges(f)
    implies
    converges_to(triangular_sum(f), double_sum(f))
} by {
    if nonneg_fn_2(f) and double_sum_converges(f) {
        // square_sum converges to double_sum(f)
        converges_to(square_sum(f), double_sum(f))

        let a = double_sum(f)

        // For any eps > 0, we need to show tail_bound(triangular_sum(f), a, big_n, eps)
        forall(eps: Real) {
            if eps.is_positive {
                // Since square_sum converges to a, there exists N
                let n: Nat satisfy {
                    tail_bound(square_sum(f), a, n, eps)
                }

                // For all m >= 2*N, we have triangular_sum(f, m) close to a
                let big_n = n + n

                forall(m: Nat) {
                    if big_n <= m {
                        // m >= 2*N, so m >= N
                        n <= m

                        // By tail_bound for square_sum, square_sum(f, m).is_close(a, eps)
                        square_sum(f, m).is_close(a, eps)

                        // We need to show: there exists k such that 2*k <= m and k >= N
                        // Since m >= 2*N, we can take k = N, and we have 2*N <= m

                        // Define k such that 2*k <= m < 2*(k+1)
                        // For simplicity, let's use k = N since m >= 2*N means 2*N <= m

                        // From our lemmas:
                        // square_sum(f, n) <= triangular_sum(f, 2*n)
                        square_sum(f, n) <= triangular_sum(f, n + n)

                        // triangular_sum is increasing for nonnegative functions
                        // We need to prove triangular_sum(f, n + n) <= triangular_sum(f, m)
                        // We'll use induction-style reasoning on the distance m - (n+n)
                        forall(j: Nat) {
                        }
                        is_increasing(triangular_sum(f))
                        define increasing_to_m(k: Nat) -> Bool {
                            n + n + k <= m
                            implies
                            triangular_sum(f, n + n) <= triangular_sum(f, n + n + k)
                        }

                        increasing_to_m(Nat.0)

                        forall(k: Nat) {
                            if increasing_to_m(k) {
                                if n + n + k.suc <= m {
                                    n + n + k < n + n + k.suc
                                    n + n + k <= n + n + k.suc
                                    n + n + k <= m
                                    triangular_sum(f, n + n + k) <= triangular_sum(f, n + n + k.suc)
                                    triangular_sum(f, n + n) <= triangular_sum(f, n + n + k)
                                    triangular_sum(f, n + n) <= triangular_sum(f, n + n + k.suc)
                                    increasing_to_m(k.suc)
                                }
                            }
                        }

                        // Apply with k = m - (n+n)
                        n + n + (m - (n + n)) = m
                        n + n <= m
                        triangular_sum(f, n + n) <= triangular_sum(f, m)
                        triangular_sum(f, n + n + (m - (n + n))) = triangular_sum(f, m)
                        triangular_sum(f, n + n) <= triangular_sum(f, n + n + (m - (n + n)))
                        n + n + (m - (n + n)) <= m
                        n + n + (m - (n + n)) <= m implies triangular_sum(f, n + n) <= triangular_sum(f, n + n + (m - (n + n)))
                        increasing_to_m(m - (n + n))
                        triangular_sum(f, n + n) <= triangular_sum(f, m)
                        square_sum(f, n) <= triangular_sum(f, m)

                        // Also triangular_sum(f, m) <= square_sum(f, m)
                        triangular_sum(f, m) <= square_sum(f, m)

                        // Since square_sum(f, n).is_close(a, eps) and square_sum(f, m).is_close(a, eps)
                        square_sum(f, n).is_close(a, eps)

                        // From is_close, we have:
                        // a - eps < square_sum(f, n) < a + eps
                        // a - eps < square_sum(f, m) < a + eps
                        // And square_sum(f, n) <= triangular_sum(f, m) <= square_sum(f, m)

                        // We need to show: a - eps < triangular_sum(f, m) < a + eps
                        // From square_sum(f, n) <= triangular_sum(f, m) and a - eps < square_sum(f, n)
                        a - eps < square_sum(f, n)
                        square_sum(f, n) <= triangular_sum(f, m)
                        a - eps < triangular_sum(f, m)

                        // From triangular_sum(f, m) <= square_sum(f, m) and square_sum(f, m) < a + eps
                        triangular_sum(f, m) <= square_sum(f, m)
                        square_sum(f, m) < a + eps
                        triangular_sum(f, m) < a + eps

                        // Therefore triangular_sum(f, m).is_close(a, eps)
                        triangular_sum(f, m).is_close(a, eps)
                    }
                }

                tail_bound(triangular_sum(f), a, big_n, eps)
            }
        }

        converges_to(triangular_sum(f), a)
        converges_to(triangular_sum(f), double_sum(f))
    }
}

/// Absolute convergence: double sum of absolute values converges.
define abs_double_sum_converges(f: (Nat, Nat) -> Real) -> Bool {
    double_sum_converges(abs_fn_2(f))
}

/// Triangular sum decomposition: f equals difference of positive and negative parts.
theorem triangular_sum_decomposition_parts(f: (Nat, Nat) -> Real, n: Nat) {
    triangular_sum(f, n) = triangular_sum(pos_part_2(f), n) - triangular_sum(neg_part_2(f), n)
} by {
    triangular_sum(f, n) = partial(tri_row_sum(f, n), n)
    triangular_sum(pos_part_2(f), n) = partial(tri_row_sum(pos_part_2(f), n), n)
    triangular_sum(neg_part_2(f), n) = partial(tri_row_sum(neg_part_2(f), n), n)

    // Show tri_row_sum decomposes for all i
    forall(i: Nat) {
        if i < n {
            // Show partial(f(i), m) = partial(pos_part_2(f)(i), m) - partial(neg_part_2(f)(i), m) for all m
            define p(m: Nat) -> Bool {
                partial(f(i), m) = partial(pos_part_2(f)(i), m) - partial(neg_part_2(f)(i), m)
            }

            p(Nat.0)

            forall(m: Nat) {
                if p(m) {
                    partial(f(i), m.suc) = partial(f(i), m) + f(i, m)
                    partial(pos_part_2(f)(i), m.suc) = partial(pos_part_2(f)(i), m) + pos_part_2(f)(i, m)
                    partial(neg_part_2(f)(i), m.suc) = partial(neg_part_2(f)(i), m) + neg_part_2(f)(i, m)

                    pos_part_2(f)(i, m) = pos_part_2(f, i, m)
                    neg_part_2(f)(i, m) = neg_part_2(f, i, m)
                    f(i, m) = pos_part_2(f, i, m) - neg_part_2(f, i, m)

                    partial(f(i), m.suc) = (partial(pos_part_2(f)(i), m) - partial(neg_part_2(f)(i), m)) + (pos_part_2(f, i, m) - neg_part_2(f, i, m))
                    partial(f(i), m.suc) = (partial(pos_part_2(f)(i), m) + pos_part_2(f, i, m)) - (partial(neg_part_2(f)(i), m) + neg_part_2(f, i, m))
                    partial(f(i), m.suc) = partial(pos_part_2(f)(i), m.suc) - partial(neg_part_2(f)(i), m.suc)
                    p(m.suc)
                }
            }

            partial(f(i), n - i) = partial(pos_part_2(f)(i), n - i) - partial(neg_part_2(f)(i), n - i)
            tri_row_sum(f, n, i) = partial(f(i), n - i)
            tri_row_sum(pos_part_2(f), n, i) = partial(pos_part_2(f)(i), n - i)
            tri_row_sum(neg_part_2(f), n, i) = partial(neg_part_2(f)(i), n - i)
            tri_row_sum(f, n, i) = tri_row_sum(pos_part_2(f), n, i) - tri_row_sum(neg_part_2(f), n, i)
        }
    }

    // Now show partial sums decompose
    define q(k: Nat) -> Bool {
        k <= n
        implies
        partial(tri_row_sum(f, n), k) = partial(tri_row_sum(pos_part_2(f), n), k) - partial(tri_row_sum(neg_part_2(f), n), k)
    }

    q(Nat.0)

    forall(k: Nat) {
        if q(k) {
            if k.suc <= n {
                k < n
                tri_row_sum(f, n, k) = tri_row_sum(pos_part_2(f), n, k) - tri_row_sum(neg_part_2(f), n, k)
                partial(tri_row_sum(f, n), k.suc) = partial(tri_row_sum(f, n), k) + tri_row_sum(f, n, k)
                partial(tri_row_sum(pos_part_2(f), n), k.suc) = partial(tri_row_sum(pos_part_2(f), n), k) + tri_row_sum(pos_part_2(f), n, k)
                partial(tri_row_sum(neg_part_2(f), n), k.suc) = partial(tri_row_sum(neg_part_2(f), n), k) + tri_row_sum(neg_part_2(f), n, k)

                partial(tri_row_sum(f, n), k) = partial(tri_row_sum(pos_part_2(f), n), k) - partial(tri_row_sum(neg_part_2(f), n), k)
                partial(tri_row_sum(f, n), k.suc) = (partial(tri_row_sum(pos_part_2(f), n), k) - partial(tri_row_sum(neg_part_2(f), n), k)) + (tri_row_sum(pos_part_2(f), n, k) - tri_row_sum(neg_part_2(f), n, k))
                partial(tri_row_sum(f, n), k.suc) = (partial(tri_row_sum(pos_part_2(f), n), k) + tri_row_sum(pos_part_2(f), n, k)) - (partial(tri_row_sum(neg_part_2(f), n), k) + tri_row_sum(neg_part_2(f), n, k))
                partial(tri_row_sum(f, n), k.suc) = partial(tri_row_sum(pos_part_2(f), n), k.suc) - partial(tri_row_sum(neg_part_2(f), n), k.suc)
                q(k.suc)
            }
        }
    }

    forall(i: Nat) {
        sub_seq(tri_row_sum(pos_part_2(f), n), tri_row_sum(neg_part_2(f), n), i) =
            tri_row_sum(pos_part_2(f), n, i) - tri_row_sum(neg_part_2(f), n, i)
        tri_row_sum(f, n, i) =
            sub_seq(tri_row_sum(pos_part_2(f), n), tri_row_sum(neg_part_2(f), n), i)
    }
    tri_row_sum(f, n) = sub_seq(tri_row_sum(pos_part_2(f), n), tri_row_sum(neg_part_2(f), n))
    partial(sub_seq(tri_row_sum(pos_part_2(f), n), tri_row_sum(neg_part_2(f), n)), n) =
        partial(tri_row_sum(pos_part_2(f), n), n) - partial(tri_row_sum(neg_part_2(f), n), n)
    partial(tri_row_sum(f, n), n) =
        partial(tri_row_sum(pos_part_2(f), n), n) - partial(tri_row_sum(neg_part_2(f), n), n)
    (n <= n) implies partial(tri_row_sum(f, n), n) = partial(tri_row_sum(pos_part_2(f), n), n) - partial(tri_row_sum(neg_part_2(f), n), n)
    q(n)
    partial(tri_row_sum(f, n), n) = partial(tri_row_sum(pos_part_2(f), n), n) - partial(tri_row_sum(neg_part_2(f), n), n)
    triangular_sum(f, n) = triangular_sum(pos_part_2(f), n) - triangular_sum(neg_part_2(f), n)
}

/// Triangular sum of positive part is bounded by triangular sum of absolute value.
theorem triangular_sum_pos_part_le_abs(f: (Nat, Nat) -> Real, n: Nat) {
    triangular_sum(pos_part_2(f), n) <= triangular_sum(abs_fn_2(f), n)
} by {
    forall(i: Nat, j: Nat) {
        pos_part_2(f, i, j) = f(i, j).max(Real.0)
        pos_part_2(f, i, j) >= Real.0
    }
    forall(i: Nat, j: Nat) {
        abs_fn_2(f, i, j) = f(i, j).abs
        f(i, j).abs >= Real.0
    }
    forall(i: Nat, j: Nat) {
        pos_part_2(f, i, j) <= abs_fn_2(f, i, j)
    }
    lte_fn_2(pos_part_2(f), abs_fn_2(f))
    triangular_sum(pos_part_2(f), n) <= triangular_sum(abs_fn_2(f), n)
}

/// Triangular sum of negative part is bounded by triangular sum of absolute value.
theorem triangular_sum_neg_part_le_abs(f: (Nat, Nat) -> Real, n: Nat) {
    triangular_sum(neg_part_2(f), n) <= triangular_sum(abs_fn_2(f), n)
} by {
    forall(i: Nat, j: Nat) {
        neg_part_2(f, i, j) = (-f(i, j)).max(Real.0)
        neg_part_2(f, i, j) >= Real.0
    }
    forall(i: Nat, j: Nat) {
        abs_fn_2(f, i, j) = f(i, j).abs
        f(i, j).abs >= Real.0
    }
    forall(i: Nat, j: Nat) {
        neg_part_2(f, i, j) <= abs_fn_2(f, i, j)
    }
    lte_fn_2(neg_part_2(f), abs_fn_2(f))
    triangular_sum(neg_part_2(f), n) <= triangular_sum(abs_fn_2(f), n)
}

/// For absolutely convergent series, the positive part has convergent double sum.
theorem abs_conv_pos_part_converges(f: (Nat, Nat) -> Real, s: Real) {
    double_image_is_supremum(rectangular_sum(abs_fn_2(f)), s)
    implies
    double_sum_converges(pos_part_2(f))
}

/// For absolutely convergent series, the negative part has convergent double sum.
theorem abs_conv_neg_part_converges(f: (Nat, Nat) -> Real, s: Real) {
    double_image_is_supremum(rectangular_sum(abs_fn_2(f)), s)
    implies
    double_sum_converges(neg_part_2(f))
}

/// For absolutely convergent series, the triangular sum converges to the double sum.
theorem triangular_sum_abs_converges_to_double_sum(f: (Nat, Nat) -> Real, s: Real, s_pos: Real, s_neg: Real) {
    double_image_is_supremum(rectangular_sum(abs_fn_2(f)), s) and
    double_image_is_supremum(rectangular_sum(pos_part_2(f)), s_pos) and
    double_image_is_supremum(rectangular_sum(neg_part_2(f)), s_neg)
    implies
    converges_to(triangular_sum(f), double_sum(f))
} by {
    if double_image_is_supremum(rectangular_sum(abs_fn_2(f)), s) and
       double_image_is_supremum(rectangular_sum(pos_part_2(f)), s_pos) and
        double_image_is_supremum(rectangular_sum(neg_part_2(f)), s_neg) {

        // Both positive and negative parts have convergent double sums
        double_sum_converges(pos_part_2(f))

        double_sum_converges(neg_part_2(f))

        // Both parts are nonnegative
        forall(i: Nat, j: Nat) {
            pos_part_2(f, i, j) >= Real.0
        }
        nonneg_fn_2(pos_part_2(f))

        forall(i: Nat, j: Nat) {
            neg_part_2(f, i, j) >= Real.0
        }
        nonneg_fn_2(neg_part_2(f))

        // Apply the nonnegative theorem to each part
        converges_to(triangular_sum(pos_part_2(f)), double_sum(pos_part_2(f)))

        converges_to(triangular_sum(neg_part_2(f)), double_sum(neg_part_2(f)))

        // triangular_sum(f, n) = triangular_sum(pos_part_2(f), n) - triangular_sum(neg_part_2(f), n)
        forall(n: Nat) {
            triangular_sum(f, n) = triangular_sum(pos_part_2(f), n) - triangular_sum(neg_part_2(f), n)
            sub_seq(triangular_sum(pos_part_2(f)), triangular_sum(neg_part_2(f)), n) =
                triangular_sum(pos_part_2(f), n) - triangular_sum(neg_part_2(f), n)
            triangular_sum(f, n) = sub_seq(triangular_sum(pos_part_2(f)), triangular_sum(neg_part_2(f)), n)
        }
        triangular_sum(f) = sub_seq(triangular_sum(pos_part_2(f)), triangular_sum(neg_part_2(f)))

        // Both triangular sums converge
        converges(triangular_sum(pos_part_2(f)))
        converges(triangular_sum(neg_part_2(f)))

        limit(triangular_sum(pos_part_2(f))) = double_sum(pos_part_2(f))
        limit(triangular_sum(neg_part_2(f))) = double_sum(neg_part_2(f))

        // Show that triangular_sum(f) = sub_seq(...) converges
        // sub_seq(a, b) = add_seq(a, neg_seq(b))
        sub_seq(triangular_sum(pos_part_2(f)), triangular_sum(neg_part_2(f))) =
            add_seq(triangular_sum(pos_part_2(f)), neg_seq(triangular_sum(neg_part_2(f))))
        converges(neg_seq(triangular_sum(neg_part_2(f))))
        converges(add_seq(triangular_sum(pos_part_2(f)), neg_seq(triangular_sum(neg_part_2(f)))))
        converges(sub_seq(triangular_sum(pos_part_2(f)), triangular_sum(neg_part_2(f))))
        converges(triangular_sum(f))

        // Apply limit_sub_seq to show the limit of the difference
        limit(sub_seq(triangular_sum(pos_part_2(f)), triangular_sum(neg_part_2(f)))) =
            limit(triangular_sum(pos_part_2(f))) - limit(triangular_sum(neg_part_2(f)))
        limit(triangular_sum(f)) = limit(triangular_sum(pos_part_2(f))) - limit(triangular_sum(neg_part_2(f)))
        limit(triangular_sum(f)) = double_sum(pos_part_2(f)) - double_sum(neg_part_2(f))

        // Show that double_sum(f) = double_sum(pos_part_2(f)) - double_sum(neg_part_2(f))
        // First show rectangular_sum(f) = sub_fn_2(rectangular_sum(pos_part_2(f)), rectangular_sum(neg_part_2(f)))
        forall(m: Nat, n: Nat) {
            rectangular_sum(f, m, n) = rectangular_sum(pos_part_2(f), m, n) - rectangular_sum(neg_part_2(f), m, n)
            sub_fn_2(rectangular_sum(pos_part_2(f)), rectangular_sum(neg_part_2(f)), m, n) =
                rectangular_sum(pos_part_2(f), m, n) - rectangular_sum(neg_part_2(f), m, n)
            rectangular_sum(f, m, n) = sub_fn_2(rectangular_sum(pos_part_2(f)), rectangular_sum(neg_part_2(f)), m, n)
        }
        rectangular_sum(f) = sub_fn_2(rectangular_sum(pos_part_2(f)), rectangular_sum(neg_part_2(f)))

        // Both pos and neg parts have convergent double sums
        double_converges(rectangular_sum(pos_part_2(f)))
        double_converges(rectangular_sum(neg_part_2(f)))

        // Apply double_limit_sub to get double_converges_to for the difference
        double_converges_to(sub_fn_2(rectangular_sum(pos_part_2(f)), rectangular_sum(neg_part_2(f))),
            double_limit(rectangular_sum(pos_part_2(f))) - double_limit(rectangular_sum(neg_part_2(f))))

        // Since rectangular_sum(f) = sub_fn_2(...), they have the same limit
        double_converges_to(rectangular_sum(f),
            double_limit(rectangular_sum(pos_part_2(f))) - double_limit(rectangular_sum(neg_part_2(f))))

        // By definition, double_sum(g) = double_limit(rectangular_sum(g)) when g converges
        double_sum(pos_part_2(f)) = double_limit(rectangular_sum(pos_part_2(f)))
        double_sum(neg_part_2(f)) = double_limit(rectangular_sum(neg_part_2(f)))
        double_sum(f) = double_limit(rectangular_sum(f))

        double_limit(rectangular_sum(f)) = double_limit(rectangular_sum(pos_part_2(f))) - double_limit(rectangular_sum(neg_part_2(f)))
        double_sum(f) = double_sum(pos_part_2(f)) - double_sum(neg_part_2(f))

        limit(triangular_sum(f)) = double_sum(f)
        converges_to(triangular_sum(f), limit(triangular_sum(f)))
        converges_to(triangular_sum(f), double_sum(f))
    }
}

/// Alias endpoint: triangular summation is additive pointwise.
theorem triangular_sum_add_distrib(f: (Nat, Nat) -> Real, g: (Nat, Nat) -> Real, n: Nat) {
    triangular_sum(add_fn_2(f, g), n) = triangular_sum(f, n) + triangular_sum(g, n)
} by {
    triangular_sum_add(f, g, n)
}

/// Alias endpoint: triangular summation commutes with left scalar multiplication.
theorem triangular_sum_scale_left(c: Real, f: (Nat, Nat) -> Real, n: Nat) {
    triangular_sum(scalar_mul_fn_2(c, f), n) = c * triangular_sum(f, n)
} by {
    triangular_sum_scale(c, f, n)
}

/// Alias endpoint: pointwise order transports to triangular finite sums.
theorem triangular_sum_le_of_lte_fn(f: (Nat, Nat) -> Real, g: (Nat, Nat) -> Real, n: Nat) {
    lte_fn_2(f, g) implies triangular_sum(f, n) <= triangular_sum(g, n)
} by {
    triangular_sum_monotone(f, g, n)
}

/// Alias endpoint: a nonnegative triangular partial sum is increasing in its cutoff.
theorem triangular_sum_step_nonneg(f: (Nat, Nat) -> Real, n: Nat) {
    nonneg_fn_2(f) implies triangular_sum(f, n) <= triangular_sum(f, n.suc)
} by {
    triangular_sum_increasing(f, n)
}

/// Triangular sums as a sequence are monotone for nonnegative functions, with a transport-oriented name.
theorem triangular_sum_seq_monotone(f: (Nat, Nat) -> Real) {
    nonneg_fn_2(f) implies is_monotone(triangular_sum_seq(f))
} by {
    triangular_sum_is_monotone(f)
}

/// Alias endpoint: triangular sums are bounded by the corresponding square sum for nonnegative terms.
theorem triangular_sum_bound_by_square(f: (Nat, Nat) -> Real, n: Nat) {
    nonneg_fn_2(f) implies triangular_sum(f, n) <= square_sum(f, n)
} by {
    triangular_sum_le_square_sum(f, n)
}

/// Alias endpoint: a square is bounded by a doubled triangular cutoff for nonnegative terms.
theorem square_sum_bound_by_triangular_double(f: (Nat, Nat) -> Real, n: Nat) {
    nonneg_fn_2(f) implies square_sum(f, n) <= triangular_sum(f, n + n)
} by {
    triangular_sum_ge_square_sum(f, n)
}

/// Alias endpoint: split a triangular cutoff into a rectangle and two shifted triangles.
theorem triangular_sum_split_rectangles(f: (Nat, Nat) -> Real, m: Nat, n: Nat) {
    triangular_sum(f, m + n) =
        rectangular_sum(f, m, n) +
        triangular_sum(shift_cols(f, n), m) +
        triangular_sum(shift_rows(f, m), n)
} by {
    triangular_sum_decomposition(f, m, n)
}
