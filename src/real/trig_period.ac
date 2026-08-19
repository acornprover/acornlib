from real.real_field import Real, zero_is_different_than_one
from real.trig import sin_zero, cos_zero, sin_neg, cos_neg
from real.trig_identities import sin_add, cos_add
from real.pi import pi, pi_over_two, cos_pi_over_two_zero, sin_pi_over_two_one,
    cos_pi_neg_one, sin_pi_zero, sin_two_pi_zero, cos_two_pi_one

numerals Real

// Cofunction identities.
//
// These follow from the addition formulas and the values (pi/2).cos = 0 and
// (pi/2).sin = 1 proved in real/pi.ac.  Parity ((-x).sin = -x.sin,
// (-x).cos = x.cos) was already proved in real/trig.ac.

/// The sine of x plus pi over two is cosine of x.
theorem sin_add_pi_over_two(x: Real) {
    (x + pi_over_two).sin = x.cos
} by {
    sin_add(x, pi_over_two)
    (x + pi_over_two).sin = x.sin * pi_over_two.cos + x.cos * pi_over_two.sin
    cos_pi_over_two_zero
    pi_over_two.cos = Real.0
    sin_pi_over_two_one
    pi_over_two.sin = Real.1
    x.sin * pi_over_two.cos + x.cos * pi_over_two.sin = x.sin * Real.0 + x.cos * Real.1
    x.sin * Real.0 = Real.0
    x.cos * Real.1 = x.cos
    x.sin * Real.0 + x.cos * Real.1 = Real.0 + x.cos
    Real.0 + x.cos = x.cos
    (x + pi_over_two).sin = x.cos
}

/// The cosine of x plus pi over two is negative sine of x.
theorem cos_add_pi_over_two(x: Real) {
    (x + pi_over_two).cos = -x.sin
} by {
    cos_add(x, pi_over_two)
    (x + pi_over_two).cos = x.cos * pi_over_two.cos - x.sin * pi_over_two.sin
    cos_pi_over_two_zero
    pi_over_two.cos = Real.0
    sin_pi_over_two_one
    pi_over_two.sin = Real.1
    x.cos * pi_over_two.cos - x.sin * pi_over_two.sin = x.cos * Real.0 - x.sin * Real.1
    x.cos * Real.0 = Real.0
    x.sin * Real.1 = x.sin
    x.cos * Real.0 - x.sin * Real.1 = Real.0 - x.sin
    Real.0 - x.sin = -x.sin
    (x + pi_over_two).cos = -x.sin
}

/// The sine of pi over two minus x is cosine of x.
theorem sin_pi_over_two_sub(x: Real) {
    (pi_over_two - x).sin = x.cos
} by {
    pi_over_two - x = pi_over_two + -x
    sin_add(pi_over_two, -x)
    (pi_over_two + -x).sin = pi_over_two.sin * (-x).cos + pi_over_two.cos * (-x).sin
    (pi_over_two - x).sin = pi_over_two.sin * (-x).cos + pi_over_two.cos * (-x).sin
    sin_pi_over_two_one
    pi_over_two.sin = Real.1
    cos_pi_over_two_zero
    pi_over_two.cos = Real.0
    cos_neg(x)
    (-x).cos = x.cos
    sin_neg(x)
    (-x).sin = -x.sin
    pi_over_two.sin * (-x).cos + pi_over_two.cos * (-x).sin = Real.1 * x.cos + Real.0 * -x.sin
    Real.1 * x.cos = x.cos
    Real.0 * -x.sin = Real.0
    Real.1 * x.cos + Real.0 * -x.sin = x.cos + Real.0
    x.cos + Real.0 = x.cos
    (pi_over_two - x).sin = x.cos
}

/// The cosine of pi over two minus x is sine of x.
theorem cos_pi_over_two_sub(x: Real) {
    (pi_over_two - x).cos = x.sin
} by {
    pi_over_two - x = pi_over_two + -x
    cos_add(pi_over_two, -x)
    (pi_over_two + -x).cos = pi_over_two.cos * (-x).cos - pi_over_two.sin * (-x).sin
    (pi_over_two - x).cos = pi_over_two.cos * (-x).cos - pi_over_two.sin * (-x).sin
    cos_pi_over_two_zero
    pi_over_two.cos = Real.0
    sin_pi_over_two_one
    pi_over_two.sin = Real.1
    cos_neg(x)
    (-x).cos = x.cos
    sin_neg(x)
    (-x).sin = -x.sin
    pi_over_two.cos * (-x).cos - pi_over_two.sin * (-x).sin = Real.0 * x.cos - Real.1 * -x.sin
    Real.0 * x.cos = Real.0
    Real.1 * -x.sin = -x.sin
    Real.0 * x.cos - Real.1 * -x.sin = Real.0 - -x.sin
    Real.0 - -x.sin = x.sin
    (pi_over_two - x).cos = x.sin
}

// Periodicity.
//
// Two pi is a period of sine and cosine, and pi is a half-period: shifting by
// pi negates both functions.  Both facts follow from the addition formulas and
// the values (2pi).cos = 1, (2pi).sin = 0, pi.cos = -1, pi.sin = 0 proved in
// real/pi.ac.

/// The sine of x plus two pi is sine of x.
theorem sin_add_two_pi(x: Real) {
    (x + (pi + pi)).sin = x.sin
} by {
    sin_add(x, pi + pi)
    (x + (pi + pi)).sin = x.sin * (pi + pi).cos + x.cos * (pi + pi).sin
    cos_two_pi_one
    (pi + pi).cos = Real.1
    sin_two_pi_zero
    (pi + pi).sin = Real.0
    x.sin * (pi + pi).cos + x.cos * (pi + pi).sin = x.sin * Real.1 + x.cos * Real.0
    x.sin * Real.1 = x.sin
    x.cos * Real.0 = Real.0
    x.sin * Real.1 + x.cos * Real.0 = x.sin + Real.0
    x.sin + Real.0 = x.sin
    (x + (pi + pi)).sin = x.sin
}

/// The cosine of x plus two pi is cosine of x.
theorem cos_add_two_pi(x: Real) {
    (x + (pi + pi)).cos = x.cos
} by {
    cos_add(x, pi + pi)
    (x + (pi + pi)).cos = x.cos * (pi + pi).cos - x.sin * (pi + pi).sin
    cos_two_pi_one
    (pi + pi).cos = Real.1
    sin_two_pi_zero
    (pi + pi).sin = Real.0
    x.cos * (pi + pi).cos - x.sin * (pi + pi).sin = x.cos * Real.1 - x.sin * Real.0
    x.cos * Real.1 = x.cos
    x.sin * Real.0 = Real.0
    x.cos * Real.1 - x.sin * Real.0 = x.cos - Real.0
    x.cos - Real.0 = x.cos
    (x + (pi + pi)).cos = x.cos
}

/// The sine of x plus pi is negative sine of x.
theorem sin_add_pi(x: Real) {
    (x + pi).sin = -x.sin
} by {
    sin_add(x, pi)
    (x + pi).sin = x.sin * pi.cos + x.cos * pi.sin
    cos_pi_neg_one
    pi.cos = -Real.1
    sin_pi_zero
    pi.sin = Real.0
    x.sin * pi.cos + x.cos * pi.sin = x.sin * -Real.1 + x.cos * Real.0
    x.sin * -Real.1 = -x.sin
    x.cos * Real.0 = Real.0
    x.sin * -Real.1 + x.cos * Real.0 = -x.sin + Real.0
    -x.sin + Real.0 = -x.sin
    (x + pi).sin = -x.sin
}

/// The cosine of x plus pi is negative cosine of x.
theorem cos_add_pi(x: Real) {
    (x + pi).cos = -x.cos
} by {
    cos_add(x, pi)
    (x + pi).cos = x.cos * pi.cos - x.sin * pi.sin
    cos_pi_neg_one
    pi.cos = -Real.1
    sin_pi_zero
    pi.sin = Real.0
    x.cos * pi.cos - x.sin * pi.sin = x.cos * -Real.1 - x.sin * Real.0
    x.cos * -Real.1 = -x.cos
    x.sin * Real.0 = Real.0
    x.cos * -Real.1 - x.sin * Real.0 = -x.cos - Real.0
    -x.cos - Real.0 = -x.cos
    (x + pi).cos = -x.cos
}

// Sine and cosine are not constant.  Minimality of the period is left as a
// comment: 2pi is the smallest positive period of sine and cosine — every
// period is an integer multiple of 2pi — but proving that needs x.sin > 0 on
// (0, pi/2), which requires the derivative-based monotonicity machinery that is
// not yet available.

/// Sine is not a constant function.
theorem sin_not_constant {
    not exists(c: Real) {
        forall(x: Real) { x.sin = c }
    }
} by {
    if exists(c: Real) {
        forall(x: Real) { x.sin = c }
    } {
        let c: Real satisfy {
            forall(x: Real) { x.sin = c }
        }
        forall(x: Real) { x.sin = c }
        sin_zero
        (Real.0).sin = Real.0
        (Real.0).sin = c
        Real.0 = c
        sin_pi_over_two_one
        pi_over_two.sin = Real.1
        pi_over_two.sin = c
        Real.1 = c
        Real.1 = Real.0
        zero_is_different_than_one
        Real.0 != Real.1
        false
    }
}

/// Cosine is not a constant function.
theorem cos_not_constant {
    not exists(c: Real) {
        forall(x: Real) { x.cos = c }
    }
} by {
    if exists(c: Real) {
        forall(x: Real) { x.cos = c }
    } {
        let c: Real satisfy {
            forall(x: Real) { x.cos = c }
        }
        forall(x: Real) { x.cos = c }
        cos_zero
        (Real.0).cos = Real.1
        (Real.0).cos = c
        Real.1 = c
        cos_pi_over_two_zero
        pi_over_two.cos = Real.0
        pi_over_two.cos = c
        Real.0 = c
        Real.0 = Real.1
        zero_is_different_than_one
        Real.0 != Real.1
        false
    }
}
