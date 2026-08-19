/// The exponential distribution with rate λ > 0.
///
/// The probability density function of the exponential law with rate λ is
///
///     f(x) = λ·e^(-λx)   for x ≥ 0,   and   f(x) = 0   for x < 0.
///
/// This file develops:
///   - the density functions and their nonnegativity and positivity,
///   - the decay bounds of e^(-λx) on the nonnegative half-line,
///   - the total mass on [0, b]: the integral of the density over [0, b] is
///     1 - e^(-λb), which tends to one as b grows,
///   - the cumulative distribution function F(x) = 1 - e^(-λx), its bounds
///     and its monotonicity,
///   - the survival function S(x) = e^(-λx) = 1 - F(x), its multiplicative
///     law S(s + t) = S(s)·S(t), and the memoryless property
///     P(X > s + t | X > s) = P(X > t),
///   - the expectation over [0, b]: the integral of t·λ·e^(-λt) over [0, b]
///     is 1/λ - e^(-λb)·(b + 1/λ), which tends to 1/λ as b grows.  (The
///     variance over [0, b] would need the second moment ∫ t²·λ·e^(-λt),
///     whose antiderivative involves a quadratic polynomial factor; it is
///     left for a follow-up, see the notes at the end of the file.)
///
/// All values are Riemann integrals over finite intervals [0, b]; the limit
/// statements are recorded in comments, since the library has no improper
/// integrals.  Integrability and value both follow from the pattern of
/// real.distribution_common: the integrand has the continuous antiderivative
/// t ↦ -e^(-λt) (or t ↦ -(t + 1/λ)·e^(-λt) for the moment), and it is
/// Lipschitz on [0, b] because its derivative is bounded there.

from real.real_field import Real
from real.gamma import neg_fn, exp_neg_fn, exp_neg_fn_derivative
from real.exp import exp_add, exp_zero, exp_pos
from real.integral_exp import exp_lte_mono, exp_continuous
from real.continuity_base import continuous
from real.continuity_sequences import identity_function_is_continuous
from real.continuity_const_mul import const_mul_left, continuous_const_mul_left
from real.continuity_composition import continuous_compose
from real.continuity_pointwise import continuous_pointwise_neg, continuous_pointwise_add
from real.calculus_api import is_derivative_fn, derivative_fn_identity, derivative_fn_const_mul, derivative_fn_compose, derivative_fn_neg, derivative_fn_mul, derivative_fn_add
from real.derivative_continuity import div_mul_cancel_denominator
from real.integral import integral, is_integrable, interval_contains, interval_contains_left, interval_contains_right, integral_additivity
from real.integral_exp import ftc2_general
from real.integral_trig import fn_integrable_gen
from real.distribution_common import derivative_bound_imp_lipschitz_on_pair, derivative_bound_imp_lipschitz_on, fn_integrable_gen_derivative_bound, integral_ftc2_derivative_bound
from real.integral_polynomial_values import integral_and_integrable_eq_on
from real.distribution_common import is_derivative_fn_both_eq
from data.basic.functions import identity_fn, compose, function_extensionality, function_eq_transport_predicate_rev, function_eq_transport_predicate
from data.basic.function_algebra import pointwise_mul, pointwise_neg, pointwise_add
from data.basic.logic import eq_true_intro
from order import lt_imp_lte, lte_trans, lt_imp_ne, lt_imp_ne_symm
from ordered_field import mul_le_mul_of_nonneg_right, zero_is_smaller_than_one
from algebra.add_ordered_group import add_le_add_right, add_le_add, neg_le_neg
from real.real_ring import real_mul_comm, mul_assoc, mul_abs, mul_nonneg, mul_le_mul_nonneg, square_nonneg, mul_pos_pos, mul_distrib_left
from real.real_base import neg_distrib, abs_gte_zero, abs_neg, gt_zero_imp_pos, pos_gt_zero
from real.derivative_trig import abs_of_nonneg
from real.real_series import triangle_ineq
from real.real_field import mul_inverse, div_mul_cancel_left
from real.continuity_pointwise_mul import continuous_pointwise_mul
from real.distribution_common import sub_nonneg_of_lte, lte_sub_of_nonneg, mul_nonneg_lte, mul_le_mul_nonneg_lte, is_derivative_fn_pointwise_add_comm

numerals Real

// ---------------------------------------------------------------------------
// The density functions
// ---------------------------------------------------------------------------

/// The exponential decay of rate λ: t ↦ e^(-λt).
define exp_decay(rate: Real, t: Real) -> Real {
    (-(rate * t)).exp
}

/// The rate-scaled density on the nonnegative half-line: t ↦ λ·e^(-λt).
define exp_rate_density(rate: Real, t: Real) -> Real {
    rate * exp_decay(rate, t)
}

/// The derivative of the rate-scaled density: t ↦ -λ²·e^(-λt).
define exp_rate_density_deriv(rate: Real, t: Real) -> Real {
    -(rate * rate) * exp_decay(rate, t)
}

/// The antiderivative of the rate-scaled density: t ↦ -e^(-λt).
define exp_rate_anti(rate: Real, t: Real) -> Real {
    -exp_decay(rate, t)
}

/// The probability density of the exponential law with rate λ: λ·e^(-λx) on
/// the nonnegative half-line and zero elsewhere.
define exponential_pdf(rate: Real, x: Real) -> Real {
    if Real.0 <= x { exp_rate_density(rate, x) } else { Real.0 }
}

// ---------------------------------------------------------------------------
// Decay bounds
// ---------------------------------------------------------------------------

/// The exponential decay is strictly positive.
theorem exp_decay_pos(rate: Real, t: Real) {
    exp_decay(rate, t) > Real.0
} by {
    exp_pos(-(rate * t))
    (-(rate * t)).exp > Real.0
    exp_decay(rate, t) = (-(rate * t)).exp
    exp_decay(rate, t) > Real.0
}

/// The exponential decay is nonnegative.
theorem exp_decay_nonneg(rate: Real, t: Real) {
    Real.0 <= exp_decay(rate, t)
} by {
    exp_decay_pos(rate, t)
    exp_decay(rate, t) > Real.0
    lt_imp_lte(Real.0, exp_decay(rate, t))
    Real.0 <= exp_decay(rate, t)
}

/// For λ > 0 and t ≥ 0 the decay e^(-λt) is at most one.
theorem exp_decay_le_one(rate: Real, t: Real) {
    rate > Real.0 and Real.0 <= t implies exp_decay(rate, t) <= Real.1
} by {
    if rate > Real.0 and Real.0 <= t {
        lt_imp_lte(Real.0, rate)
        Real.0 <= rate
        mul_nonneg_lte(rate, t)
        Real.0 <= rate * t
        neg_le_neg(Real.0, rate * t)
        -(rate * t) <= -Real.0
        -Real.0 = Real.0
        -(rate * t) <= Real.0
        exp_lte_mono(-(rate * t), Real.0)
        (-(rate * t)).exp <= (Real.0).exp
        exp_zero
        (Real.0).exp = Real.1
        (-(rate * t)).exp <= Real.1
        exp_decay(rate, t) = (-(rate * t)).exp
        exp_decay(rate, t) <= Real.1
    }
}

// ---------------------------------------------------------------------------
// Nonnegativity of the density
// ---------------------------------------------------------------------------

/// The rate-scaled density is nonnegative.
theorem exp_rate_density_nonneg(rate: Real, t: Real) {
    Real.0 <= rate implies Real.0 <= exp_rate_density(rate, t)
} by {
    if Real.0 <= rate {
        exp_decay_nonneg(rate, t)
        Real.0 <= exp_decay(rate, t)
        mul_nonneg_lte(rate, exp_decay(rate, t))
        Real.0 <= rate * exp_decay(rate, t)
        exp_rate_density(rate, t) = rate * exp_decay(rate, t)
        Real.0 <= exp_rate_density(rate, t)
    }
}

/// The rate-scaled density is at most the rate on the nonnegative half-line.
theorem exp_rate_density_le_rate(rate: Real, t: Real) {
    rate > Real.0 and Real.0 <= t implies exp_rate_density(rate, t) <= rate
} by {
    if rate > Real.0 and Real.0 <= t {
        exp_decay_le_one(rate, t)
        exp_decay(rate, t) <= Real.1
        lt_imp_lte(Real.0, rate)
        Real.0 <= rate
        mul_le_mul_of_nonneg_right(exp_decay(rate, t), Real.1, rate)
        exp_decay(rate, t) * rate <= Real.1 * rate
        real_mul_comm(rate, exp_decay(rate, t))
        exp_decay(rate, t) * rate = rate * exp_decay(rate, t)
        Real.1 * rate = rate
        rate * exp_decay(rate, t) <= rate
        exp_rate_density(rate, t) = rate * exp_decay(rate, t)
        exp_rate_density(rate, t) <= rate
    }
}

/// The density of the exponential law is nonnegative.
theorem exponential_pdf_nonneg(rate: Real, x: Real) {
    rate > Real.0 implies Real.0 <= exponential_pdf(rate, x)
} by {
    if rate > Real.0 {
        lt_imp_lte(Real.0, rate)
        Real.0 <= rate
        if Real.0 <= x {
            exponential_pdf(rate, x) = exp_rate_density(rate, x)
            exp_rate_density_nonneg(rate, x)
            Real.0 <= exp_rate_density(rate, x)
            Real.0 <= exponential_pdf(rate, x)
        }
        if not (Real.0 <= x) {
            exponential_pdf(rate, x) = Real.0
            Real.0 <= exponential_pdf(rate, x)
        }
        Real.0 <= x or not (Real.0 <= x)
        Real.0 <= exponential_pdf(rate, x)
    }
}

/// The density of the exponential law is strictly positive on the
/// nonnegative half-line.
theorem exponential_pdf_pos(rate: Real, x: Real) {
    rate > Real.0 and Real.0 <= x implies exponential_pdf(rate, x) > Real.0
} by {
    if rate > Real.0 and Real.0 <= x {
        exponential_pdf(rate, x) = exp_rate_density(rate, x)
        exp_decay_pos(rate, x)
        exp_decay(rate, x) > Real.0
        gt_zero_imp_pos(exp_decay(rate, x))
        exp_decay(rate, x).is_positive
        gt_zero_imp_pos(rate)
        rate.is_positive
        mul_pos_pos(rate, exp_decay(rate, x))
        (rate * exp_decay(rate, x)).is_positive
        pos_gt_zero(rate * exp_decay(rate, x))
        rate * exp_decay(rate, x) > Real.0
        exp_rate_density(rate, x) = rate * exp_decay(rate, x)
        exp_rate_density(rate, x) > Real.0
        exponential_pdf(rate, x) > Real.0
    }
}

/// The density of the exponential law vanishes on the negative half-line.
theorem exponential_pdf_zero_neg(rate: Real, x: Real) {
    not (Real.0 <= x) implies exponential_pdf(rate, x) = Real.0
} by {
    if not (Real.0 <= x) {
        exponential_pdf(rate, x) = Real.0
    }
}

/// On the nonnegative half-line the density agrees with λ·e^(-λx).
theorem exponential_pdf_eq_density(rate: Real, x: Real) {
    Real.0 <= x implies exponential_pdf(rate, x) = exp_rate_density(rate, x)
} by {
    if Real.0 <= x {
        exponential_pdf(rate, x) = exp_rate_density(rate, x)
    }
}

// ---------------------------------------------------------------------------
// Continuity
// ---------------------------------------------------------------------------

/// The pointwise negation x ↦ -x is continuous.
theorem neg_fn_continuous {
    continuous(neg_fn)
} by {
    identity_function_is_continuous
    continuous(identity_fn[Real])
    continuous_pointwise_neg(identity_fn[Real])
    continuous(pointwise_neg(identity_fn[Real]))
    forall(x: Real) {
        neg_fn(x) = -x
        pointwise_neg(identity_fn[Real], x) = -identity_fn[Real](x)
        identity_fn[Real](x) = x
        pointwise_neg(identity_fn[Real], x) = -x
        neg_fn(x) = pointwise_neg(identity_fn[Real], x)
    }
    function_extensionality(neg_fn, pointwise_neg(identity_fn[Real]))
    define neg_cont_pred(h: Real -> Real) -> Bool {
        continuous(h)
    }
    neg_cont_pred(pointwise_neg(identity_fn[Real]))
    function_eq_transport_predicate_rev(neg_cont_pred, neg_fn,
        pointwise_neg(identity_fn[Real]))
    continuous(neg_fn)
}

/// The function t ↦ e^(-t) is continuous.
theorem exp_neg_fn_continuous {
    continuous(exp_neg_fn)
} by {
    exp_continuous
    continuous(Real.exp)
    neg_fn_continuous
    continuous(neg_fn)
    continuous_compose(neg_fn, Real.exp)
    continuous(compose(Real.exp, neg_fn))
    forall(x: Real) {
        exp_neg_fn(x) = (neg_fn(x)).exp
        compose(Real.exp, neg_fn, x) = (neg_fn(x)).exp
        exp_neg_fn(x) = compose(Real.exp, neg_fn, x)
    }
    function_extensionality(exp_neg_fn, compose(Real.exp, neg_fn))
    define neg_exp_cont_pred(h: Real -> Real) -> Bool {
        continuous(h)
    }
    neg_exp_cont_pred(compose(Real.exp, neg_fn))
    function_eq_transport_predicate_rev(neg_exp_cont_pred, exp_neg_fn,
        compose(Real.exp, neg_fn))
    continuous(exp_neg_fn)
}

/// The decay t ↦ e^(-λt) is continuous.
theorem exp_decay_continuous(rate: Real) {
    continuous(exp_decay(rate))
} by {
    identity_function_is_continuous
    continuous(identity_fn[Real])
    continuous_const_mul_left(rate, identity_fn[Real])
    continuous(const_mul_left(rate, identity_fn[Real]))
    exp_neg_fn_continuous
    continuous(exp_neg_fn)
    continuous_compose(const_mul_left(rate, identity_fn[Real]), exp_neg_fn)
    continuous(compose(exp_neg_fn, const_mul_left(rate, identity_fn[Real])))
    forall(x: Real) {
        exp_decay(rate, x) = (-(rate * x)).exp
        compose(exp_neg_fn, const_mul_left(rate, identity_fn[Real]), x) =
            exp_neg_fn(const_mul_left(rate, identity_fn[Real], x))
        const_mul_left(rate, identity_fn[Real], x) = rate * identity_fn[Real](x)
        identity_fn[Real](x) = x
        const_mul_left(rate, identity_fn[Real], x) = rate * x
        exp_neg_fn(rate * x) = (neg_fn(rate * x)).exp
        neg_fn(rate * x) = -(rate * x)
        exp_neg_fn(rate * x) = (-(rate * x)).exp
        exp_decay(rate, x) = compose(exp_neg_fn, const_mul_left(rate, identity_fn[Real]), x)
    }
    function_extensionality(exp_decay(rate),
        compose(exp_neg_fn, const_mul_left(rate, identity_fn[Real])))
    define decay_cont_pred(h: Real -> Real) -> Bool {
        continuous(h)
    }
    decay_cont_pred(compose(exp_neg_fn, const_mul_left(rate, identity_fn[Real])))
    function_eq_transport_predicate_rev(decay_cont_pred, exp_decay(rate),
        compose(exp_neg_fn, const_mul_left(rate, identity_fn[Real])))
    continuous(exp_decay(rate))
}

/// The rate-scaled density is continuous.
theorem exp_rate_density_continuous(rate: Real) {
    continuous(exp_rate_density(rate))
} by {
    exp_decay_continuous(rate)
    continuous(exp_decay(rate))
    continuous_const_mul_left(rate, exp_decay(rate))
    continuous(const_mul_left(rate, exp_decay(rate)))
    forall(x: Real) {
        exp_rate_density(rate, x) = rate * exp_decay(rate, x)
        const_mul_left(rate, exp_decay(rate), x) = rate * exp_decay(rate, x)
        exp_rate_density(rate, x) = const_mul_left(rate, exp_decay(rate), x)
    }
    function_extensionality(exp_rate_density(rate),
        const_mul_left(rate, exp_decay(rate)))
    define density_cont_pred(h: Real -> Real) -> Bool {
        continuous(h)
    }
    density_cont_pred(const_mul_left(rate, exp_decay(rate)))
    function_eq_transport_predicate_rev(density_cont_pred, exp_rate_density(rate),
        const_mul_left(rate, exp_decay(rate)))
    continuous(exp_rate_density(rate))
}

/// The antiderivative t ↦ -e^(-λt) is continuous.
theorem exp_rate_anti_continuous(rate: Real) {
    continuous(exp_rate_anti(rate))
} by {
    exp_decay_continuous(rate)
    continuous(exp_decay(rate))
    continuous_pointwise_neg(exp_decay(rate))
    continuous(pointwise_neg(exp_decay(rate)))
    forall(x: Real) {
        exp_rate_anti(rate, x) = -exp_decay(rate, x)
        pointwise_neg(exp_decay(rate), x) = -exp_decay(rate, x)
        exp_rate_anti(rate, x) = pointwise_neg(exp_decay(rate), x)
    }
    function_extensionality(exp_rate_anti(rate), pointwise_neg(exp_decay(rate)))
    define anti_cont_pred(h: Real -> Real) -> Bool {
        continuous(h)
    }
    anti_cont_pred(pointwise_neg(exp_decay(rate)))
    function_eq_transport_predicate_rev(anti_cont_pred, exp_rate_anti(rate),
        pointwise_neg(exp_decay(rate)))
    continuous(exp_rate_anti(rate))
}

// ---------------------------------------------------------------------------
// Derivatives
// ---------------------------------------------------------------------------

/// The linear scaling t ↦ λ·t has derivative t ↦ λ.
theorem rate_scale_derivative(rate: Real) {
    is_derivative_fn(const_mul_left(rate, identity_fn[Real]),
        const_mul_left(rate, constant[Real, Real](Real.1)))
} by {
    derivative_fn_identity
    is_derivative_fn(identity_fn[Real], constant[Real, Real](Real.1))
    derivative_fn_const_mul(rate, identity_fn[Real], constant[Real, Real](Real.1))
    is_derivative_fn(const_mul_left(rate, identity_fn[Real]),
        const_mul_left(rate, constant[Real, Real](Real.1)))
}

/// The decay t ↦ e^(-λt) has derivative t ↦ -λ·e^(-λt).
theorem exp_decay_derivative(rate: Real) {
    is_derivative_fn(exp_decay(rate), pointwise_neg(exp_rate_density(rate)))
} by {
    exp_neg_fn_derivative
    is_derivative_fn(exp_neg_fn, pointwise_neg(exp_neg_fn))
    rate_scale_derivative(rate)
    is_derivative_fn(const_mul_left(rate, identity_fn[Real]),
        const_mul_left(rate, constant[Real, Real](Real.1)))
    derivative_fn_compose(const_mul_left(rate, identity_fn[Real]), exp_neg_fn,
        const_mul_left(rate, constant[Real, Real](Real.1)), pointwise_neg(exp_neg_fn))
    is_derivative_fn(compose(exp_neg_fn, const_mul_left(rate, identity_fn[Real])),
        pointwise_mul(compose(pointwise_neg(exp_neg_fn), const_mul_left(rate, identity_fn[Real])),
            const_mul_left(rate, constant[Real, Real](Real.1))))
    forall(x: Real) {
        compose(exp_neg_fn, const_mul_left(rate, identity_fn[Real]), x) =
            exp_neg_fn(const_mul_left(rate, identity_fn[Real], x))
        const_mul_left(rate, identity_fn[Real], x) = rate * identity_fn[Real](x)
        identity_fn[Real](x) = x
        const_mul_left(rate, identity_fn[Real], x) = rate * x
        exp_neg_fn(rate * x) = (neg_fn(rate * x)).exp
        neg_fn(rate * x) = -(rate * x)
        exp_neg_fn(rate * x) = (-(rate * x)).exp
        exp_decay(rate, x) = (-(rate * x)).exp
        exp_neg_fn(rate * x) = exp_decay(rate, x)
        pointwise_neg(exp_neg_fn, rate * x) = -exp_neg_fn(rate * x)
        pointwise_neg(exp_neg_fn, rate * x) = -exp_decay(rate, x)
        compose(pointwise_neg(exp_neg_fn), const_mul_left(rate, identity_fn[Real]), x) =
            pointwise_neg(exp_neg_fn, const_mul_left(rate, identity_fn[Real], x))
        const_mul_left(rate, identity_fn[Real], x) = rate * x
        compose(pointwise_neg(exp_neg_fn), const_mul_left(rate, identity_fn[Real]), x) =
            pointwise_neg(exp_neg_fn, rate * x)
        compose(pointwise_neg(exp_neg_fn), const_mul_left(rate, identity_fn[Real]), x) =
            -exp_decay(rate, x)
        const_mul_left(rate, constant[Real, Real](Real.1), x) =
            rate * constant[Real, Real](Real.1, x)
        constant[Real, Real](Real.1, x) = Real.1
        const_mul_left(rate, constant[Real, Real](Real.1), x) = rate
        pointwise_mul(compose(pointwise_neg(exp_neg_fn), const_mul_left(rate, identity_fn[Real])),
            const_mul_left(rate, constant[Real, Real](Real.1)), x) =
            compose(pointwise_neg(exp_neg_fn), const_mul_left(rate, identity_fn[Real]), x) *
            const_mul_left(rate, constant[Real, Real](Real.1), x)
        pointwise_mul(compose(pointwise_neg(exp_neg_fn), const_mul_left(rate, identity_fn[Real])),
            const_mul_left(rate, constant[Real, Real](Real.1)), x) = (-exp_decay(rate, x)) * rate
        (-exp_decay(rate, x)) * rate = -(exp_decay(rate, x) * rate)
        exp_rate_density(rate, x) = rate * exp_decay(rate, x)
        real_mul_comm(exp_decay(rate, x), rate)
        exp_decay(rate, x) * rate = rate * exp_decay(rate, x)
        -(exp_decay(rate, x) * rate) = -exp_rate_density(rate, x)
        pointwise_mul(compose(pointwise_neg(exp_neg_fn), const_mul_left(rate, identity_fn[Real])),
            const_mul_left(rate, constant[Real, Real](Real.1)), x) =
            -exp_rate_density(rate, x)
        pointwise_neg(exp_rate_density(rate), x) = -exp_rate_density(rate, x)
        pointwise_mul(compose(pointwise_neg(exp_neg_fn), const_mul_left(rate, identity_fn[Real])),
            const_mul_left(rate, constant[Real, Real](Real.1)), x) =
            pointwise_neg(exp_rate_density(rate), x)
    }
    function_extensionality(
        pointwise_mul(compose(pointwise_neg(exp_neg_fn), const_mul_left(rate, identity_fn[Real])),
            const_mul_left(rate, constant[Real, Real](Real.1))),
        pointwise_neg(exp_rate_density(rate)))
    forall(x: Real) {
        exp_decay(rate, x) = (-(rate * x)).exp
        compose(exp_neg_fn, const_mul_left(rate, identity_fn[Real]), x) =
            exp_neg_fn(const_mul_left(rate, identity_fn[Real], x))
        const_mul_left(rate, identity_fn[Real], x) = rate * identity_fn[Real](x)
        identity_fn[Real](x) = x
        const_mul_left(rate, identity_fn[Real], x) = rate * x
        exp_neg_fn(rate * x) = (neg_fn(rate * x)).exp
        neg_fn(rate * x) = -(rate * x)
        exp_neg_fn(rate * x) = (-(rate * x)).exp
        exp_decay(rate, x) = compose(exp_neg_fn, const_mul_left(rate, identity_fn[Real]), x)
    }
    function_extensionality(exp_decay(rate),
        compose(exp_neg_fn, const_mul_left(rate, identity_fn[Real])))
    is_derivative_fn_both_eq(
        compose(exp_neg_fn, const_mul_left(rate, identity_fn[Real])),
        exp_decay(rate),
        pointwise_mul(compose(pointwise_neg(exp_neg_fn), const_mul_left(rate, identity_fn[Real])),
            const_mul_left(rate, constant[Real, Real](Real.1))),
        pointwise_neg(exp_rate_density(rate)))
    is_derivative_fn(exp_decay(rate), pointwise_neg(exp_rate_density(rate)))
}

/// The antiderivative t ↦ -e^(-λt) of the rate-scaled density.
theorem exp_rate_anti_derivative(rate: Real) {
    is_derivative_fn(exp_rate_anti(rate), exp_rate_density(rate))
} by {
    exp_decay_derivative(rate)
    is_derivative_fn(exp_decay(rate), pointwise_neg(exp_rate_density(rate)))
    derivative_fn_neg(exp_decay(rate), pointwise_neg(exp_rate_density(rate)))
    is_derivative_fn(pointwise_neg(exp_decay(rate)),
        pointwise_neg(pointwise_neg(exp_rate_density(rate))))
    forall(x: Real) {
        pointwise_neg(pointwise_neg(exp_rate_density(rate)), x) =
            -pointwise_neg(exp_rate_density(rate), x)
        pointwise_neg(exp_rate_density(rate), x) = -exp_rate_density(rate, x)
        -(-exp_rate_density(rate, x)) = exp_rate_density(rate, x)
        pointwise_neg(pointwise_neg(exp_rate_density(rate)), x) = exp_rate_density(rate, x)
    }
    function_extensionality(pointwise_neg(pointwise_neg(exp_rate_density(rate))),
        exp_rate_density(rate))
    define anti_deriv_pred(h: Real -> Real) -> Bool {
        is_derivative_fn(pointwise_neg(exp_decay(rate)), h)
    }
    anti_deriv_pred(pointwise_neg(pointwise_neg(exp_rate_density(rate))))
    function_eq_transport_predicate_rev(anti_deriv_pred, exp_rate_density(rate),
        pointwise_neg(pointwise_neg(exp_rate_density(rate))))
    is_derivative_fn(pointwise_neg(exp_decay(rate)), exp_rate_density(rate))
    forall(x: Real) {
        exp_rate_anti(rate, x) = -exp_decay(rate, x)
        pointwise_neg(exp_decay(rate), x) = -exp_decay(rate, x)
        exp_rate_anti(rate, x) = pointwise_neg(exp_decay(rate), x)
    }
    function_extensionality(exp_rate_anti(rate), pointwise_neg(exp_decay(rate)))
    define anti_fn_pred(h: Real -> Real) -> Bool {
        is_derivative_fn(h, exp_rate_density(rate))
    }
    anti_fn_pred(pointwise_neg(exp_decay(rate)))
    function_eq_transport_predicate_rev(anti_fn_pred, exp_rate_anti(rate),
        pointwise_neg(exp_decay(rate)))
    is_derivative_fn(exp_rate_anti(rate), exp_rate_density(rate))
}

/// The rate-scaled density has derivative t ↦ -λ²·e^(-λt).
theorem exp_rate_density_derivative(rate: Real) {
    is_derivative_fn(exp_rate_density(rate), exp_rate_density_deriv(rate))
} by {
    exp_decay_derivative(rate)
    is_derivative_fn(exp_decay(rate), pointwise_neg(exp_rate_density(rate)))
    derivative_fn_const_mul(rate, exp_decay(rate), pointwise_neg(exp_rate_density(rate)))
    is_derivative_fn(const_mul_left(rate, exp_decay(rate)),
        const_mul_left(rate, pointwise_neg(exp_rate_density(rate))))
    forall(x: Real) {
        const_mul_left(rate, pointwise_neg(exp_rate_density(rate)), x) =
            rate * pointwise_neg(exp_rate_density(rate), x)
        pointwise_neg(exp_rate_density(rate), x) = -exp_rate_density(rate, x)
        rate * (-exp_rate_density(rate, x)) = -(rate * exp_rate_density(rate, x))
        exp_rate_density(rate, x) = rate * exp_decay(rate, x)
        rate * exp_rate_density(rate, x) = rate * (rate * exp_decay(rate, x))
        mul_assoc(rate, rate, exp_decay(rate, x))
        rate * (rate * exp_decay(rate, x)) = (rate * rate) * exp_decay(rate, x)
        rate * exp_rate_density(rate, x) = (rate * rate) * exp_decay(rate, x)
        -(rate * exp_rate_density(rate, x)) = -((rate * rate) * exp_decay(rate, x))
        exp_rate_density_deriv(rate, x) = -(rate * rate) * exp_decay(rate, x)
        const_mul_left(rate, pointwise_neg(exp_rate_density(rate)), x) =
            -(rate * exp_rate_density(rate, x))
        const_mul_left(rate, pointwise_neg(exp_rate_density(rate)), x) =
            exp_rate_density_deriv(rate, x)
    }
    function_extensionality(const_mul_left(rate, pointwise_neg(exp_rate_density(rate))),
        exp_rate_density_deriv(rate))
    define density_deriv_pred(h: Real -> Real) -> Bool {
        is_derivative_fn(const_mul_left(rate, exp_decay(rate)), h)
    }
    density_deriv_pred(const_mul_left(rate, pointwise_neg(exp_rate_density(rate))))
    function_eq_transport_predicate_rev(density_deriv_pred, exp_rate_density_deriv(rate),
        const_mul_left(rate, pointwise_neg(exp_rate_density(rate))))
    is_derivative_fn(const_mul_left(rate, exp_decay(rate)), exp_rate_density_deriv(rate))
    forall(x: Real) {
        exp_rate_density(rate, x) = rate * exp_decay(rate, x)
        const_mul_left(rate, exp_decay(rate), x) = rate * exp_decay(rate, x)
        exp_rate_density(rate, x) = const_mul_left(rate, exp_decay(rate), x)
    }
    function_extensionality(exp_rate_density(rate), const_mul_left(rate, exp_decay(rate)))
    define density_fn_pred(h: Real -> Real) -> Bool {
        is_derivative_fn(h, exp_rate_density_deriv(rate))
    }
    density_fn_pred(const_mul_left(rate, exp_decay(rate)))
    function_eq_transport_predicate_rev(density_fn_pred, exp_rate_density(rate),
        const_mul_left(rate, exp_decay(rate)))
    is_derivative_fn(exp_rate_density(rate), exp_rate_density_deriv(rate))
}

// ---------------------------------------------------------------------------
// The derivative bound of the density on [0, b]
// ---------------------------------------------------------------------------

/// The derivative of the density is bounded by λ² on [a, b] for a ≥ 0.
theorem exp_rate_density_deriv_bound(rate: Real, a: Real, b: Real, z: Real) {
    rate > Real.0 and Real.0 <= a and a <= b and interval_contains(a, b, z)
    implies exp_rate_density_deriv(rate, z).abs <= rate * rate
} by {
    if rate > Real.0 and Real.0 <= a and a <= b and interval_contains(a, b, z) {
        lt_imp_lte(Real.0, rate)
        Real.0 <= rate
        mul_nonneg_lte(rate, rate)
        Real.0 <= rate * rate
        interval_contains_left(a, b, z)
        a <= z
        lte_trans(Real.0, a, z)
        Real.0 <= z
        exp_rate_density_deriv(rate, z) = -(rate * rate) * exp_decay(rate, z)
        abs_neg((rate * rate) * exp_decay(rate, z))
        (-((rate * rate) * exp_decay(rate, z))).abs = ((rate * rate) * exp_decay(rate, z)).abs
        exp_rate_density_deriv(rate, z).abs = ((rate * rate) * exp_decay(rate, z)).abs
        mul_abs(rate * rate, exp_decay(rate, z))
        ((rate * rate) * exp_decay(rate, z)).abs = (rate * rate).abs * exp_decay(rate, z).abs
        abs_of_nonneg(rate * rate)
        (rate * rate).abs = rate * rate
        exp_decay_nonneg(rate, z)
        Real.0 <= exp_decay(rate, z)
        abs_of_nonneg(exp_decay(rate, z))
        exp_decay(rate, z).abs = exp_decay(rate, z)
        exp_rate_density_deriv(rate, z).abs = (rate * rate) * exp_decay(rate, z)
        exp_decay_le_one(rate, z)
        exp_decay(rate, z) <= Real.1
        mul_le_mul_of_nonneg_right(exp_decay(rate, z), Real.1, rate * rate)
        exp_decay(rate, z) * (rate * rate) <= Real.1 * (rate * rate)
        real_mul_comm(rate * rate, exp_decay(rate, z))
        exp_decay(rate, z) * (rate * rate) = (rate * rate) * exp_decay(rate, z)
        Real.1 * (rate * rate) = rate * rate
        (rate * rate) * exp_decay(rate, z) <= rate * rate
        exp_rate_density_deriv(rate, z).abs <= rate * rate
    }
}

// ---------------------------------------------------------------------------
// Bounds of the density on [a, b]
// ---------------------------------------------------------------------------

/// The density is bounded below by zero on [a, b] for a ≥ 0.
theorem exp_rate_density_lower_bound_on(rate: Real, a: Real, b: Real, t: Real) {
    rate > Real.0 and Real.0 <= a and a <= b and interval_contains(a, b, t)
    implies Real.0 <= exp_rate_density(rate, t)
} by {
    if rate > Real.0 and Real.0 <= a and a <= b and interval_contains(a, b, t) {
        lt_imp_lte(Real.0, rate)
        Real.0 <= rate
        exp_rate_density_nonneg(rate, t)
        Real.0 <= exp_rate_density(rate, t)
    }
}

/// The density is bounded above by λ on [a, b] for a ≥ 0.
theorem exp_rate_density_upper_bound_on(rate: Real, a: Real, b: Real, t: Real) {
    rate > Real.0 and Real.0 <= a and a <= b and interval_contains(a, b, t)
    implies exp_rate_density(rate, t) <= rate
} by {
    if rate > Real.0 and Real.0 <= a and a <= b and interval_contains(a, b, t) {
        interval_contains_left(a, b, t)
        a <= t
        lte_trans(Real.0, a, t)
        Real.0 <= t
        exp_rate_density_le_rate(rate, t)
        exp_rate_density(rate, t) <= rate
    }
}

// ---------------------------------------------------------------------------
// Integrability and the total mass on [0, b]
// ---------------------------------------------------------------------------

/// The density λ·e^(-λt) is integrable on [a, b] for 0 ≤ a ≤ b.
theorem exp_rate_density_integrable(rate: Real, a: Real, b: Real) {
    rate > Real.0 and Real.0 <= a and a <= b implies
    is_integrable(exp_rate_density(rate), a, b)
} by {
    if rate > Real.0 and Real.0 <= a and a <= b {
        forall(z: Real) {
            if interval_contains(a, b, z) {
                exp_rate_density_deriv_bound(rate, a, b, z)
                exp_rate_density_deriv(rate, z).abs <= rate * rate
            }
        }
        forall(t: Real) {
            if interval_contains(a, b, t) {
                exp_rate_density_lower_bound_on(rate, a, b, t)
                Real.0 <= exp_rate_density(rate, t)
            }
        }
        forall(t: Real) {
            if interval_contains(a, b, t) {
                exp_rate_density_upper_bound_on(rate, a, b, t)
                exp_rate_density(rate, t) <= rate
            }
        }
        lt_imp_lte(Real.0, rate)
        Real.0 <= rate
        mul_nonneg_lte(rate, rate)
        Real.0 <= rate * rate
        exp_rate_anti_continuous(rate)
        continuous(exp_rate_anti(rate))
        exp_rate_anti_derivative(rate)
        is_derivative_fn(exp_rate_anti(rate), exp_rate_density(rate))
        exp_rate_density_derivative(rate)
        is_derivative_fn(exp_rate_density(rate), exp_rate_density_deriv(rate))
        exp_rate_density_continuous(rate)
        continuous(exp_rate_density(rate))
        eq_true_intro(forall(z: Real) {
            interval_contains(a, b, z) implies exp_rate_density_deriv(rate, z).abs <= rate * rate
        })
        (forall(z: Real) {
            interval_contains(a, b, z) implies exp_rate_density_deriv(rate, z).abs <= rate * rate
        }) = true
        eq_true_intro(forall(t: Real) {
            interval_contains(a, b, t) implies Real.0 <= exp_rate_density(rate, t)
        })
        (forall(t: Real) {
            interval_contains(a, b, t) implies Real.0 <= exp_rate_density(rate, t)
        }) = true
        eq_true_intro(forall(t: Real) {
            interval_contains(a, b, t) implies exp_rate_density(rate, t) <= rate
        })
        (forall(t: Real) {
            interval_contains(a, b, t) implies exp_rate_density(rate, t) <= rate
        }) = true
        eq_true_intro(continuous(exp_rate_anti(rate)))
        (continuous(exp_rate_anti(rate))) = true
        eq_true_intro(is_derivative_fn(exp_rate_anti(rate), exp_rate_density(rate)))
        (is_derivative_fn(exp_rate_anti(rate), exp_rate_density(rate))) = true
        eq_true_intro(is_derivative_fn(exp_rate_density(rate), exp_rate_density_deriv(rate)))
        (is_derivative_fn(exp_rate_density(rate), exp_rate_density_deriv(rate))) = true
        eq_true_intro(continuous(exp_rate_density(rate)))
        (continuous(exp_rate_density(rate))) = true
        a <= b and Real.0 <= rate * rate and
            continuous(exp_rate_anti(rate)) and
            is_derivative_fn(exp_rate_anti(rate), exp_rate_density(rate)) and
            is_derivative_fn(exp_rate_density(rate), exp_rate_density_deriv(rate)) and
            continuous(exp_rate_density(rate)) and
            (forall(z: Real) {
                interval_contains(a, b, z) implies exp_rate_density_deriv(rate, z).abs <= rate * rate
            }) and
            (forall(t: Real) {
                interval_contains(a, b, t) implies Real.0 <= exp_rate_density(rate, t)
            }) and
            (forall(t: Real) {
                interval_contains(a, b, t) implies exp_rate_density(rate, t) <= rate
            })
        fn_integrable_gen_derivative_bound(
            exp_rate_density(rate), exp_rate_anti(rate), exp_rate_density_deriv(rate),
            a, b, rate * rate, Real.0, rate)
        is_integrable(exp_rate_density(rate), a, b)
    }
}

/// The total mass of the exponential density over [0, b] is 1 - e^(-λb).
theorem exp_rate_mass_interval(rate: Real, b: Real) {
    rate > Real.0 and Real.0 <= b implies
    integral(exp_rate_density(rate), Real.0, b) = Real.1 - exp_decay(rate, b)
} by {
    if rate > Real.0 and Real.0 <= b {
        forall(z: Real) {
            if interval_contains(Real.0, b, z) {
                exp_rate_density_deriv_bound(rate, Real.0, b, z)
                exp_rate_density_deriv(rate, z).abs <= rate * rate
            }
        }
        forall(t: Real) {
            if interval_contains(Real.0, b, t) {
                exp_rate_density_lower_bound_on(rate, Real.0, b, t)
                Real.0 <= exp_rate_density(rate, t)
            }
        }
        forall(t: Real) {
            if interval_contains(Real.0, b, t) {
                exp_rate_density_upper_bound_on(rate, Real.0, b, t)
                exp_rate_density(rate, t) <= rate
            }
        }
        lt_imp_lte(Real.0, rate)
        Real.0 <= rate
        mul_nonneg_lte(rate, rate)
        Real.0 <= rate * rate
        exp_rate_anti_continuous(rate)
        continuous(exp_rate_anti(rate))
        exp_rate_anti_derivative(rate)
        is_derivative_fn(exp_rate_anti(rate), exp_rate_density(rate))
        exp_rate_density_derivative(rate)
        is_derivative_fn(exp_rate_density(rate), exp_rate_density_deriv(rate))
        exp_rate_density_continuous(rate)
        continuous(exp_rate_density(rate))
        eq_true_intro(forall(z: Real) {
            interval_contains(Real.0, b, z) implies exp_rate_density_deriv(rate, z).abs <= rate * rate
        })
        (forall(z: Real) {
            interval_contains(Real.0, b, z) implies exp_rate_density_deriv(rate, z).abs <= rate * rate
        }) = true
        eq_true_intro(forall(t: Real) {
            interval_contains(Real.0, b, t) implies Real.0 <= exp_rate_density(rate, t)
        })
        (forall(t: Real) {
            interval_contains(Real.0, b, t) implies Real.0 <= exp_rate_density(rate, t)
        }) = true
        eq_true_intro(forall(t: Real) {
            interval_contains(Real.0, b, t) implies exp_rate_density(rate, t) <= rate
        })
        (forall(t: Real) {
            interval_contains(Real.0, b, t) implies exp_rate_density(rate, t) <= rate
        }) = true
        eq_true_intro(continuous(exp_rate_anti(rate)))
        (continuous(exp_rate_anti(rate))) = true
        eq_true_intro(is_derivative_fn(exp_rate_anti(rate), exp_rate_density(rate)))
        (is_derivative_fn(exp_rate_anti(rate), exp_rate_density(rate))) = true
        eq_true_intro(is_derivative_fn(exp_rate_density(rate), exp_rate_density_deriv(rate)))
        (is_derivative_fn(exp_rate_density(rate), exp_rate_density_deriv(rate))) = true
        eq_true_intro(continuous(exp_rate_density(rate)))
        (continuous(exp_rate_density(rate))) = true
        Real.0 <= b and Real.0 <= rate * rate and
            continuous(exp_rate_anti(rate)) and
            is_derivative_fn(exp_rate_anti(rate), exp_rate_density(rate)) and
            is_derivative_fn(exp_rate_density(rate), exp_rate_density_deriv(rate)) and
            continuous(exp_rate_density(rate)) and
            (forall(z: Real) {
                interval_contains(Real.0, b, z) implies exp_rate_density_deriv(rate, z).abs <= rate * rate
            }) and
            (forall(t: Real) {
                interval_contains(Real.0, b, t) implies Real.0 <= exp_rate_density(rate, t)
            }) and
            (forall(t: Real) {
                interval_contains(Real.0, b, t) implies exp_rate_density(rate, t) <= rate
            })
        integral_ftc2_derivative_bound(
            exp_rate_density(rate), exp_rate_anti(rate), exp_rate_density_deriv(rate),
            Real.0, b, rate * rate, Real.0, rate)
        is_integrable(exp_rate_density(rate), Real.0, b) and
            integral(exp_rate_density(rate), Real.0, b) =
                exp_rate_anti(rate, b) - exp_rate_anti(rate, Real.0)
        integral(exp_rate_density(rate), Real.0, b) =
            exp_rate_anti(rate, b) - exp_rate_anti(rate, Real.0)
        exp_rate_anti(rate, b) = -exp_decay(rate, b)
        exp_rate_anti(rate, Real.0) = -exp_decay(rate, Real.0)
        rate * Real.0 = Real.0
        -(rate * Real.0) = Real.0
        exp_decay(rate, Real.0) = (-(rate * Real.0)).exp
        exp_decay(rate, Real.0) = (Real.0).exp
        exp_zero
        (Real.0).exp = Real.1
        exp_decay(rate, Real.0) = Real.1
        exp_rate_anti(rate, Real.0) = -Real.1
        exp_rate_anti(rate, b) - exp_rate_anti(rate, Real.0) =
            -exp_decay(rate, b) - (-Real.1)
        -exp_decay(rate, b) - (-Real.1) = Real.1 - exp_decay(rate, b)
        integral(exp_rate_density(rate), Real.0, b) = Real.1 - exp_decay(rate, b)
    }
}

// ---------------------------------------------------------------------------
// The cumulative distribution function
// ---------------------------------------------------------------------------

/// The cumulative distribution function of the exponential law with rate λ:
/// the mass of [0, x].
define exponential_cdf(rate: Real, x: Real) -> Real {
    integral(exponential_pdf(rate), Real.0, x)
}

/// On [0, x] the density agrees with λ·e^(-λt).
theorem exponential_pdf_eq_density_on(rate: Real, x: Real) {
    Real.0 <= x implies
    forall(t: Real) {
        interval_contains(Real.0, x, t) implies exponential_pdf(rate, t) = exp_rate_density(rate, t)
    }
} by {
    if Real.0 <= x {
        forall(t: Real) {
            if interval_contains(Real.0, x, t) {
                interval_contains_left(Real.0, x, t)
                Real.0 <= t
                exponential_pdf_eq_density(rate, t)
                exponential_pdf(rate, t) = exp_rate_density(rate, t)
            }
        }
    }
}

/// On [0, x] the density is bounded between zero and λ.
theorem exponential_pdf_bounds_on(rate: Real, x: Real, t: Real) {
    rate > Real.0 and Real.0 <= x and interval_contains(Real.0, x, t)
    implies (Real.0 <= exponential_pdf(rate, t) and exponential_pdf(rate, t) <= rate)
} by {
    if rate > Real.0 and Real.0 <= x and interval_contains(Real.0, x, t) {
        interval_contains_left(Real.0, x, t)
        Real.0 <= t
        exponential_pdf_eq_density(rate, t)
        exponential_pdf(rate, t) = exp_rate_density(rate, t)
        exp_rate_density_nonneg(rate, t)
        Real.0 <= exp_rate_density(rate, t)
        Real.0 <= exponential_pdf(rate, t)
        exp_rate_density_le_rate(rate, t)
        exp_rate_density(rate, t) <= rate
        exponential_pdf(rate, t) <= rate
        Real.0 <= exponential_pdf(rate, t) and exponential_pdf(rate, t) <= rate
    }
}

/// The density of the exponential law is integrable on [0, x] for x ≥ 0.
theorem exponential_pdf_integrable(rate: Real, x: Real) {
    rate > Real.0 and Real.0 <= x implies is_integrable(exponential_pdf(rate), Real.0, x)
} by {
    if rate > Real.0 and Real.0 <= x {
        exp_rate_density_integrable(rate, Real.0, x)
        is_integrable(exp_rate_density(rate), Real.0, x)
        exponential_pdf_eq_density_on(rate, x)
        forall(t: Real) {
            if interval_contains(Real.0, x, t) {
                exponential_pdf(rate, t) = exp_rate_density(rate, t)
            }
        }
        forall(t: Real) {
            if interval_contains(Real.0, x, t) {
                exponential_pdf_bounds_on(rate, x, t)
                Real.0 <= exponential_pdf(rate, t) and exponential_pdf(rate, t) <= rate
                Real.0 <= exponential_pdf(rate, t)
            }
        }
        forall(t: Real) {
            if interval_contains(Real.0, x, t) {
                exponential_pdf_bounds_on(rate, x, t)
                Real.0 <= exponential_pdf(rate, t) and exponential_pdf(rate, t) <= rate
                exponential_pdf(rate, t) <= rate
            }
        }
        eq_true_intro(forall(t: Real) {
            interval_contains(Real.0, x, t) implies exponential_pdf(rate, t) = exp_rate_density(rate, t)
        })
        (forall(t: Real) {
            interval_contains(Real.0, x, t) implies exponential_pdf(rate, t) = exp_rate_density(rate, t)
        }) = true
        eq_true_intro(forall(t: Real) {
            interval_contains(Real.0, x, t) implies Real.0 <= exponential_pdf(rate, t)
        })
        (forall(t: Real) {
            interval_contains(Real.0, x, t) implies Real.0 <= exponential_pdf(rate, t)
        }) = true
        eq_true_intro(forall(t: Real) {
            interval_contains(Real.0, x, t) implies exponential_pdf(rate, t) <= rate
        })
        (forall(t: Real) {
            interval_contains(Real.0, x, t) implies exponential_pdf(rate, t) <= rate
        }) = true
        Real.0 <= x and is_integrable(exp_rate_density(rate), Real.0, x) and
            (forall(t: Real) {
                interval_contains(Real.0, x, t) implies exponential_pdf(rate, t) = exp_rate_density(rate, t)
            }) and
            (forall(t: Real) {
                interval_contains(Real.0, x, t) implies Real.0 <= exponential_pdf(rate, t)
            }) and
            (forall(t: Real) {
                interval_contains(Real.0, x, t) implies exponential_pdf(rate, t) <= rate
            })
        integral_and_integrable_eq_on(exponential_pdf(rate), exp_rate_density(rate),
            Real.0, x, Real.0, rate)
        is_integrable(exponential_pdf(rate), Real.0, x)
    }
}

/// The CDF of the exponential law at x ≥ 0 is 1 - e^(-λx).
theorem exponential_cdf_value(rate: Real, x: Real) {
    rate > Real.0 and Real.0 <= x implies
    exponential_cdf(rate, x) = Real.1 - exp_decay(rate, x)
} by {
    if rate > Real.0 and Real.0 <= x {
        exp_rate_density_integrable(rate, Real.0, x)
        is_integrable(exp_rate_density(rate), Real.0, x)
        exponential_pdf_eq_density_on(rate, x)
        forall(t: Real) {
            if interval_contains(Real.0, x, t) {
                exponential_pdf(rate, t) = exp_rate_density(rate, t)
            }
        }
        forall(t: Real) {
            if interval_contains(Real.0, x, t) {
                exponential_pdf_bounds_on(rate, x, t)
                Real.0 <= exponential_pdf(rate, t) and exponential_pdf(rate, t) <= rate
                Real.0 <= exponential_pdf(rate, t)
            }
        }
        forall(t: Real) {
            if interval_contains(Real.0, x, t) {
                exponential_pdf_bounds_on(rate, x, t)
                Real.0 <= exponential_pdf(rate, t) and exponential_pdf(rate, t) <= rate
                exponential_pdf(rate, t) <= rate
            }
        }
        eq_true_intro(forall(t: Real) {
            interval_contains(Real.0, x, t) implies exponential_pdf(rate, t) = exp_rate_density(rate, t)
        })
        (forall(t: Real) {
            interval_contains(Real.0, x, t) implies exponential_pdf(rate, t) = exp_rate_density(rate, t)
        }) = true
        eq_true_intro(forall(t: Real) {
            interval_contains(Real.0, x, t) implies Real.0 <= exponential_pdf(rate, t)
        })
        (forall(t: Real) {
            interval_contains(Real.0, x, t) implies Real.0 <= exponential_pdf(rate, t)
        }) = true
        eq_true_intro(forall(t: Real) {
            interval_contains(Real.0, x, t) implies exponential_pdf(rate, t) <= rate
        })
        (forall(t: Real) {
            interval_contains(Real.0, x, t) implies exponential_pdf(rate, t) <= rate
        }) = true
        Real.0 <= x and is_integrable(exp_rate_density(rate), Real.0, x) and
            (forall(t: Real) {
                interval_contains(Real.0, x, t) implies exponential_pdf(rate, t) = exp_rate_density(rate, t)
            }) and
            (forall(t: Real) {
                interval_contains(Real.0, x, t) implies Real.0 <= exponential_pdf(rate, t)
            }) and
            (forall(t: Real) {
                interval_contains(Real.0, x, t) implies exponential_pdf(rate, t) <= rate
            })
        integral_and_integrable_eq_on(exponential_pdf(rate), exp_rate_density(rate),
            Real.0, x, Real.0, rate)
        is_integrable(exponential_pdf(rate), Real.0, x) and
            integral(exponential_pdf(rate), Real.0, x) =
                integral(exp_rate_density(rate), Real.0, x)
        integral(exponential_pdf(rate), Real.0, x) =
            integral(exp_rate_density(rate), Real.0, x)
        exponential_cdf(rate, x) = integral(exponential_pdf(rate), Real.0, x)
        exponential_cdf(rate, x) = integral(exp_rate_density(rate), Real.0, x)
        exp_rate_mass_interval(rate, x)
        integral(exp_rate_density(rate), Real.0, x) = Real.1 - exp_decay(rate, x)
        exponential_cdf(rate, x) = Real.1 - exp_decay(rate, x)
    }
}

/// The CDF of the exponential law is nonnegative.
theorem exponential_cdf_nonneg(rate: Real, x: Real) {
    rate > Real.0 and Real.0 <= x implies Real.0 <= exponential_cdf(rate, x)
} by {
    if rate > Real.0 and Real.0 <= x {
        exponential_cdf_value(rate, x)
        exponential_cdf(rate, x) = Real.1 - exp_decay(rate, x)
        exp_decay_le_one(rate, x)
        exp_decay(rate, x) <= Real.1
        sub_nonneg_of_lte(exp_decay(rate, x), Real.1)
        Real.0 <= Real.1 - exp_decay(rate, x)
        Real.0 <= exponential_cdf(rate, x)
    }
}

/// The CDF of the exponential law is at most one.
theorem exponential_cdf_le_one(rate: Real, x: Real) {
    rate > Real.0 and Real.0 <= x implies exponential_cdf(rate, x) <= Real.1
} by {
    if rate > Real.0 and Real.0 <= x {
        exponential_cdf_value(rate, x)
        exponential_cdf(rate, x) = Real.1 - exp_decay(rate, x)
        exp_decay_nonneg(rate, x)
        Real.0 <= exp_decay(rate, x)
        lte_sub_of_nonneg(exp_decay(rate, x), Real.1)
        Real.1 - exp_decay(rate, x) <= Real.1
        exponential_cdf(rate, x) <= Real.1
    }
}

/// The CDF of the exponential law is nondecreasing on the nonnegative
/// half-line: 0 ≤ x ≤ y implies F(x) ≤ F(y).
theorem exponential_cdf_mono(rate: Real, x: Real, y: Real) {
    rate > Real.0 and Real.0 <= x and x <= y implies
    exponential_cdf(rate, x) <= exponential_cdf(rate, y)
} by {
    if rate > Real.0 and Real.0 <= x and x <= y {
        exponential_pdf_integrable(rate, x)
        is_integrable(exponential_pdf(rate), Real.0, x)
        lte_trans(Real.0, x, y)
        Real.0 <= y
        exponential_pdf_integrable(rate, y)
        is_integrable(exponential_pdf(rate), Real.0, y)
        exponential_pdf_eq_density_on(rate, y)
        forall(t: Real) {
            if interval_contains(Real.0, y, t) {
            exponential_pdf(rate, t) = exp_rate_density(rate, t)
            }
        }
        // is_integrable(pdf, x, y): bridge from density integrability on [x, y]
        exp_rate_density_integrable(rate, x, y)
        is_integrable(exp_rate_density(rate), x, y)
        forall(t: Real) {
            if interval_contains(x, y, t) {
                interval_contains_left(x, y, t)
                x <= t
                // t in [x, y] implies 0 <= t
                lte_trans(Real.0, x, t)
                Real.0 <= t
                exponential_pdf_eq_density(rate, t)
                exponential_pdf(rate, t) = exp_rate_density(rate, t)
            }
        }
        forall(t: Real) {
            if interval_contains(x, y, t) {
                interval_contains_left(x, y, t)
                x <= t
                lte_trans(Real.0, x, t)
                Real.0 <= t
                exponential_pdf_eq_density(rate, t)
                exponential_pdf(rate, t) = exp_rate_density(rate, t)
            }
        }
        forall(t: Real) {
            if interval_contains(x, y, t) {
                exponential_pdf_nonneg(rate, t)
                Real.0 <= exponential_pdf(rate, t)
            }
        }
        forall(t: Real) {
            if interval_contains(x, y, t) {
                interval_contains_left(x, y, t)
                x <= t
                lte_trans(Real.0, x, t)
                Real.0 <= t
                exponential_pdf_eq_density(rate, t)
                exponential_pdf(rate, t) = exp_rate_density(rate, t)
                exp_rate_density_le_rate(rate, t)
                exp_rate_density(rate, t) <= rate
                exponential_pdf(rate, t) <= rate
            }
        }
        eq_true_intro(forall(t: Real) {
            interval_contains(x, y, t) implies exponential_pdf(rate, t) = exp_rate_density(rate, t)
        })
        (forall(t: Real) {
            interval_contains(x, y, t) implies exponential_pdf(rate, t) = exp_rate_density(rate, t)
        }) = true
        eq_true_intro(forall(t: Real) {
            interval_contains(x, y, t) implies Real.0 <= exponential_pdf(rate, t)
        })
        (forall(t: Real) {
            interval_contains(x, y, t) implies Real.0 <= exponential_pdf(rate, t)
        }) = true
        eq_true_intro(forall(t: Real) {
            interval_contains(x, y, t) implies exponential_pdf(rate, t) <= rate
        })
        (forall(t: Real) {
            interval_contains(x, y, t) implies exponential_pdf(rate, t) <= rate
        }) = true
        x <= y and is_integrable(exp_rate_density(rate), x, y) and
            (forall(t: Real) {
                interval_contains(x, y, t) implies exponential_pdf(rate, t) = exp_rate_density(rate, t)
            }) and
            (forall(t: Real) {
                interval_contains(x, y, t) implies Real.0 <= exponential_pdf(rate, t)
            }) and
            (forall(t: Real) {
                interval_contains(x, y, t) implies exponential_pdf(rate, t) <= rate
            })
        integral_and_integrable_eq_on(exponential_pdf(rate), exp_rate_density(rate),
            x, y, Real.0, rate)
        is_integrable(exponential_pdf(rate), x, y)
        integral_additivity(exponential_pdf(rate), Real.0, x, y)
        integral(exponential_pdf(rate), Real.0, y) =
            integral(exponential_pdf(rate), Real.0, x) + integral(exponential_pdf(rate), x, y)
        exponential_cdf(rate, y) = integral(exponential_pdf(rate), Real.0, y)
        exponential_cdf(rate, x) = integral(exponential_pdf(rate), Real.0, x)
        exponential_cdf(rate, y) = exponential_cdf(rate, x) + integral(exponential_pdf(rate), x, y)
        // the mass over [x, y] is e^(-λx) - e^(-λy) >= 0
        forall(z: Real) {
            if interval_contains(x, y, z) {
                exp_rate_density_deriv_bound(rate, x, y, z)
                exp_rate_density_deriv(rate, z).abs <= rate * rate
            }
        }
        forall(t: Real) {
            if interval_contains(x, y, t) {
                exp_rate_density_lower_bound_on(rate, x, y, t)
                Real.0 <= exp_rate_density(rate, t)
            }
        }
        forall(t: Real) {
            if interval_contains(x, y, t) {
                exp_rate_density_upper_bound_on(rate, x, y, t)
                exp_rate_density(rate, t) <= rate
            }
        }
        lt_imp_lte(Real.0, rate)
        Real.0 <= rate
        mul_nonneg_lte(rate, rate)
        Real.0 <= rate * rate
        exp_rate_anti_continuous(rate)
        continuous(exp_rate_anti(rate))
        exp_rate_anti_derivative(rate)
        is_derivative_fn(exp_rate_anti(rate), exp_rate_density(rate))
        exp_rate_density_derivative(rate)
        is_derivative_fn(exp_rate_density(rate), exp_rate_density_deriv(rate))
        exp_rate_density_continuous(rate)
        continuous(exp_rate_density(rate))
        eq_true_intro(forall(z: Real) {
            interval_contains(x, y, z) implies exp_rate_density_deriv(rate, z).abs <= rate * rate
        })
        (forall(z: Real) {
            interval_contains(x, y, z) implies exp_rate_density_deriv(rate, z).abs <= rate * rate
        }) = true
        eq_true_intro(forall(t: Real) {
            interval_contains(x, y, t) implies Real.0 <= exp_rate_density(rate, t)
        })
        (forall(t: Real) {
            interval_contains(x, y, t) implies Real.0 <= exp_rate_density(rate, t)
        }) = true
        eq_true_intro(forall(t: Real) {
            interval_contains(x, y, t) implies exp_rate_density(rate, t) <= rate
        })
        (forall(t: Real) {
            interval_contains(x, y, t) implies exp_rate_density(rate, t) <= rate
        }) = true
        eq_true_intro(continuous(exp_rate_anti(rate)))
        (continuous(exp_rate_anti(rate))) = true
        eq_true_intro(is_derivative_fn(exp_rate_anti(rate), exp_rate_density(rate)))
        (is_derivative_fn(exp_rate_anti(rate), exp_rate_density(rate))) = true
        eq_true_intro(is_derivative_fn(exp_rate_density(rate), exp_rate_density_deriv(rate)))
        (is_derivative_fn(exp_rate_density(rate), exp_rate_density_deriv(rate))) = true
        eq_true_intro(continuous(exp_rate_density(rate)))
        (continuous(exp_rate_density(rate))) = true
        x <= y and Real.0 <= rate * rate and
            continuous(exp_rate_anti(rate)) and
            is_derivative_fn(exp_rate_anti(rate), exp_rate_density(rate)) and
            is_derivative_fn(exp_rate_density(rate), exp_rate_density_deriv(rate)) and
            continuous(exp_rate_density(rate)) and
            (forall(z: Real) {
                interval_contains(x, y, z) implies exp_rate_density_deriv(rate, z).abs <= rate * rate
            }) and
            (forall(t: Real) {
                interval_contains(x, y, t) implies Real.0 <= exp_rate_density(rate, t)
            }) and
            (forall(t: Real) {
                interval_contains(x, y, t) implies exp_rate_density(rate, t) <= rate
            })
        integral_ftc2_derivative_bound(
            exp_rate_density(rate), exp_rate_anti(rate), exp_rate_density_deriv(rate),
            x, y, rate * rate, Real.0, rate)
        is_integrable(exp_rate_density(rate), x, y) and
            integral(exp_rate_density(rate), x, y) =
                exp_rate_anti(rate, y) - exp_rate_anti(rate, x)
        integral(exp_rate_density(rate), x, y) =
            exp_rate_anti(rate, y) - exp_rate_anti(rate, x)
        exp_rate_anti(rate, y) = -exp_decay(rate, y)
        exp_rate_anti(rate, x) = -exp_decay(rate, x)
        exp_rate_anti(rate, y) - exp_rate_anti(rate, x) =
            -exp_decay(rate, y) - (-exp_decay(rate, x))
        -exp_decay(rate, y) - (-exp_decay(rate, x)) = exp_decay(rate, x) - exp_decay(rate, y)
        integral(exp_rate_density(rate), x, y) = exp_decay(rate, x) - exp_decay(rate, y)
        integral(exponential_pdf(rate), x, y) =
            integral(exp_rate_density(rate), x, y)
        integral(exponential_pdf(rate), x, y) = exp_decay(rate, x) - exp_decay(rate, y)
        // monotonicity of the decay: x <= y gives e^(-λx) >= e^(-λy)
        lt_imp_lte(Real.0, rate)
        Real.0 <= rate
        mul_le_mul_of_nonneg_right(x, y, rate)
        x * rate <= y * rate
        real_mul_comm(rate, x)
        x * rate = rate * x
        real_mul_comm(rate, y)
        y * rate = rate * y
        rate * x <= rate * y
        neg_le_neg(rate * x, rate * y)
        -(rate * y) <= -(rate * x)
        exp_lte_mono(-(rate * y), -(rate * x))
        (-(rate * y)).exp <= (-(rate * x)).exp
        exp_decay(rate, y) = (-(rate * y)).exp
        exp_decay(rate, x) = (-(rate * x)).exp
        exp_decay(rate, y) <= exp_decay(rate, x)
        sub_nonneg_of_lte(exp_decay(rate, y), exp_decay(rate, x))
        Real.0 <= exp_decay(rate, x) - exp_decay(rate, y)
        // F(y) = F(x) + (e^(-λx) - e^(-λy)) >= F(x)
        exponential_cdf(rate, x) + integral(exponential_pdf(rate), x, y) =
            exponential_cdf(rate, y)
        integral(exponential_pdf(rate), x, y) = exp_decay(rate, x) - exp_decay(rate, y)
        exponential_cdf(rate, x) + (exp_decay(rate, x) - exp_decay(rate, y)) =
            exponential_cdf(rate, y)
        add_le_add_right(Real.0, exp_decay(rate, x) - exp_decay(rate, y), exponential_cdf(rate, x))
        Real.0 + exponential_cdf(rate, x) <= (exp_decay(rate, x) - exp_decay(rate, y)) + exponential_cdf(rate, x)
        Real.0 + exponential_cdf(rate, x) = exponential_cdf(rate, x)
        exponential_cdf(rate, x) <= exponential_cdf(rate, y)
    }
}

// ---------------------------------------------------------------------------
// The survival function and the memoryless property
// ---------------------------------------------------------------------------

/// The survival function S(x) = e^(-λx) = P(X > x) of the exponential law.
define exponential_survival(rate: Real, x: Real) -> Real {
    exp_decay(rate, x)
}

/// The survival function is the complement of the CDF.
theorem exponential_survival_complement(rate: Real, x: Real) {
    rate > Real.0 and Real.0 <= x implies
    exponential_survival(rate, x) = Real.1 - exponential_cdf(rate, x)
} by {
    if rate > Real.0 and Real.0 <= x {
        exponential_survival(rate, x) = exp_decay(rate, x)
        exponential_cdf_value(rate, x)
        exponential_cdf(rate, x) = Real.1 - exp_decay(rate, x)
        Real.1 - exponential_cdf(rate, x) = Real.1 - (Real.1 - exp_decay(rate, x))
        Real.1 - (Real.1 - exp_decay(rate, x)) = exp_decay(rate, x)
        Real.1 - exponential_cdf(rate, x) = exp_decay(rate, x)
        exponential_survival(rate, x) = Real.1 - exponential_cdf(rate, x)
    }
}

/// The survival function is strictly positive.
theorem exponential_survival_pos(rate: Real, x: Real) {
    exponential_survival(rate, x) > Real.0
} by {
    exp_decay_pos(rate, x)
    exp_decay(rate, x) > Real.0
    exponential_survival(rate, x) = exp_decay(rate, x)
    exponential_survival(rate, x) > Real.0
}

/// The survival function is multiplicative: S(s + t) = S(s)·S(t).
theorem exponential_survival_multiplicative(rate: Real, s: Real, t: Real) {
    exponential_survival(rate, s + t) = exponential_survival(rate, s) * exponential_survival(rate, t)
} by {
    exponential_survival(rate, s + t) = exp_decay(rate, s + t)
    exp_decay(rate, s + t) = (-(rate * (s + t))).exp
    exp_decay(rate, s) = (-(rate * s)).exp
    exp_decay(rate, t) = (-(rate * t)).exp
    real_mul_comm(rate, s)
    rate * s = s * rate
    real_mul_comm(rate, t)
    rate * t = t * rate
    mul_distrib_left(rate, s, t)
    rate * (s + t) = rate * s + rate * t
    -(rate * (s + t)) = -(rate * s + rate * t)
    neg_distrib(rate * s, rate * t)
    -(rate * s + rate * t) = -(rate * s) + -(rate * t)
    exp_add(-(rate * s), -(rate * t))
    (-(rate * s) + -(rate * t)).exp = (-(rate * s)).exp * (-(rate * t)).exp
    (-(rate * (s + t))).exp = (-(rate * s)).exp * (-(rate * t)).exp
    exp_decay(rate, s + t) = (-(rate * s)).exp * (-(rate * t)).exp
    exponential_survival(rate, s) = exp_decay(rate, s)
    exponential_survival(rate, t) = exp_decay(rate, t)
    exponential_survival(rate, s) * exponential_survival(rate, t) =
        exp_decay(rate, s) * exp_decay(rate, t)
    exp_decay(rate, s + t) = exponential_survival(rate, s) * exponential_survival(rate, t)
    exponential_survival(rate, s + t) =
        exponential_survival(rate, s) * exponential_survival(rate, t)
}

/// The exponential law is memoryless:
/// P(X > s + t | X > s) = P(X > t), i.e. S(s + t) / S(s) = S(t).
theorem exponential_memoryless(rate: Real, s: Real, t: Real) {
    rate > Real.0 implies
    exponential_survival(rate, s + t) / exponential_survival(rate, s) =
        exponential_survival(rate, t)
} by {
    if rate > Real.0 {
        exponential_survival_multiplicative(rate, s, t)
        exponential_survival(rate, s + t) =
            exponential_survival(rate, s) * exponential_survival(rate, t)
        exponential_survival_pos(rate, s)
        exponential_survival(rate, s) > Real.0
        lt_imp_ne(Real.0, exponential_survival(rate, s))
        Real.0 != exponential_survival(rate, s)
        exponential_survival(rate, s) != Real.0
        div_mul_cancel_left(exponential_survival(rate, s), exponential_survival(rate, t))
        (exponential_survival(rate, s) * exponential_survival(rate, t)) /
            exponential_survival(rate, s) = exponential_survival(rate, t)
        exponential_survival(rate, s + t) / exponential_survival(rate, s) =
            (exponential_survival(rate, s) * exponential_survival(rate, t)) /
                exponential_survival(rate, s)
        exponential_survival(rate, s + t) / exponential_survival(rate, s) =
            exponential_survival(rate, t)
    }
}

// ---------------------------------------------------------------------------
// The expectation on [0, b]
// ---------------------------------------------------------------------------

/// The moment integrand t ↦ t·λ·e^(-λt).
define exponential_moment_integrand(rate: Real, t: Real) -> Real {
    t * exp_rate_density(rate, t)
}

/// The derivative of the moment integrand: t ↦ λ·e^(-λt)·(1 - λ·t).
define exponential_moment_deriv(rate: Real, t: Real) -> Real {
    exp_rate_density(rate, t) * (Real.1 - rate * t)
}

/// The antiderivative of the moment integrand: t ↦ -(t + 1/λ)·e^(-λt).
define exponential_moment_anti(rate: Real, t: Real) -> Real {
    -exp_decay(rate, t) * (t + Real.1 / rate)
}

/// The shift t ↦ t + 1/λ has derivative t ↦ 1.
theorem moment_shift_derivative(rate: Real) {
    is_derivative_fn(
        pointwise_add(identity_fn[Real], constant[Real, Real](Real.1 / rate)),
        constant[Real, Real](Real.1))
} by {
    derivative_fn_identity
    is_derivative_fn(identity_fn[Real], constant[Real, Real](Real.1))
    derivative_fn_add(identity_fn[Real], constant[Real, Real](Real.1 / rate),
        constant[Real, Real](Real.1), constant[Real, Real](Real.0))
    is_derivative_fn(
        pointwise_add(identity_fn[Real], constant[Real, Real](Real.1 / rate)),
        pointwise_add(constant[Real, Real](Real.1), constant[Real, Real](Real.0)))
    forall(x: Real) {
        pointwise_add(constant[Real, Real](Real.1), constant[Real, Real](Real.0), x) =
            constant[Real, Real](Real.1, x) + constant[Real, Real](Real.0, x)
        constant[Real, Real](Real.1, x) = Real.1
        constant[Real, Real](Real.0, x) = Real.0
        constant[Real, Real](Real.1, x) + constant[Real, Real](Real.0, x) = Real.1
        pointwise_add(constant[Real, Real](Real.1), constant[Real, Real](Real.0), x) = Real.1
        constant[Real, Real](Real.1, x) = Real.1
        pointwise_add(constant[Real, Real](Real.1), constant[Real, Real](Real.0), x) =
            constant[Real, Real](Real.1, x)
    }
    function_extensionality(
        pointwise_add(constant[Real, Real](Real.1), constant[Real, Real](Real.0)),
        constant[Real, Real](Real.1))
    define shift_deriv_pred(h: Real -> Real) -> Bool {
        is_derivative_fn(
            pointwise_add(identity_fn[Real], constant[Real, Real](Real.1 / rate)), h)
    }
    shift_deriv_pred(pointwise_add(constant[Real, Real](Real.1), constant[Real, Real](Real.0)))
    function_eq_transport_predicate_rev(shift_deriv_pred,
        constant[Real, Real](Real.1),
        pointwise_add(constant[Real, Real](Real.1), constant[Real, Real](Real.0)))
    is_derivative_fn(
        pointwise_add(identity_fn[Real], constant[Real, Real](Real.1 / rate)),
        constant[Real, Real](Real.1))
}

/// The antiderivative of the moment integrand.
theorem exponential_moment_anti_derivative(rate: Real) {
    rate > Real.0 implies
    is_derivative_fn(pointwise_neg(pointwise_mul(exp_decay(rate),
        pointwise_add(identity_fn[Real], constant[Real, Real](Real.1 / rate)))),
        exponential_moment_integrand(rate))
} by {
    if rate > Real.0 {
    exp_decay_derivative(rate)
    is_derivative_fn(exp_decay(rate), pointwise_neg(exp_rate_density(rate)))
    moment_shift_derivative(rate)
    is_derivative_fn(
        pointwise_add(identity_fn[Real], constant[Real, Real](Real.1 / rate)),
        constant[Real, Real](Real.1))
    eq_true_intro(is_derivative_fn(exp_decay(rate), pointwise_neg(exp_rate_density(rate))))
    (is_derivative_fn(exp_decay(rate), pointwise_neg(exp_rate_density(rate)))) = true
    eq_true_intro(is_derivative_fn(
        pointwise_add(identity_fn[Real], constant[Real, Real](Real.1 / rate)),
        constant[Real, Real](Real.1)))
    (is_derivative_fn(
        pointwise_add(identity_fn[Real], constant[Real, Real](Real.1 / rate)),
        constant[Real, Real](Real.1))) = true
    derivative_fn_mul(exp_decay(rate),
        pointwise_add(identity_fn[Real], constant[Real, Real](Real.1 / rate)),
        pointwise_neg(exp_rate_density(rate)), constant[Real, Real](Real.1))
    is_derivative_fn(
        pointwise_mul(exp_decay(rate),
            pointwise_add(identity_fn[Real], constant[Real, Real](Real.1 / rate))),
        pointwise_add(
            pointwise_mul(exp_decay(rate), constant[Real, Real](Real.1)),
            pointwise_mul(pointwise_add(identity_fn[Real], constant[Real, Real](Real.1 / rate)),
                pointwise_neg(exp_rate_density(rate)))))
    is_derivative_fn_pointwise_add_comm(
        pointwise_mul(exp_decay(rate),
            pointwise_add(identity_fn[Real], constant[Real, Real](Real.1 / rate))),
        pointwise_mul(exp_decay(rate), constant[Real, Real](Real.1)),
        pointwise_mul(pointwise_add(identity_fn[Real], constant[Real, Real](Real.1 / rate)),
            pointwise_neg(exp_rate_density(rate))))
    is_derivative_fn(
        pointwise_mul(exp_decay(rate),
            pointwise_add(identity_fn[Real], constant[Real, Real](Real.1 / rate))),
        pointwise_add(
            pointwise_mul(pointwise_neg(exp_rate_density(rate)),
                pointwise_add(identity_fn[Real], constant[Real, Real](Real.1 / rate))),
            pointwise_mul(exp_decay(rate), constant[Real, Real](Real.1))))
    forall(x: Real) {
        pointwise_mul(pointwise_neg(exp_rate_density(rate)),
            pointwise_add(identity_fn[Real], constant[Real, Real](Real.1 / rate)), x) =
            pointwise_neg(exp_rate_density(rate), x) *
                pointwise_add(identity_fn[Real], constant[Real, Real](Real.1 / rate), x)
        pointwise_neg(exp_rate_density(rate), x) = -exp_rate_density(rate, x)
        pointwise_add(identity_fn[Real], constant[Real, Real](Real.1 / rate), x) =
            identity_fn[Real](x) + constant[Real, Real](Real.1 / rate, x)
        identity_fn[Real](x) = x
        constant[Real, Real](Real.1 / rate, x) = Real.1 / rate
        pointwise_add(identity_fn[Real], constant[Real, Real](Real.1 / rate), x) =
            x + Real.1 / rate
        pointwise_mul(pointwise_neg(exp_rate_density(rate)),
            pointwise_add(identity_fn[Real], constant[Real, Real](Real.1 / rate)), x) =
            -exp_rate_density(rate, x) * (x + Real.1 / rate)
        pointwise_mul(exp_decay(rate), constant[Real, Real](Real.1), x) =
            exp_decay(rate, x) * constant[Real, Real](Real.1, x)
        constant[Real, Real](Real.1, x) = Real.1
        pointwise_mul(exp_decay(rate), constant[Real, Real](Real.1), x) =
            exp_decay(rate, x)
        pointwise_add(
            pointwise_mul(pointwise_neg(exp_rate_density(rate)),
                pointwise_add(identity_fn[Real], constant[Real, Real](Real.1 / rate))),
            pointwise_mul(exp_decay(rate), constant[Real, Real](Real.1)), x) =
            pointwise_mul(pointwise_neg(exp_rate_density(rate)),
                pointwise_add(identity_fn[Real], constant[Real, Real](Real.1 / rate)), x) +
            pointwise_mul(exp_decay(rate), constant[Real, Real](Real.1), x)
        pointwise_mul(pointwise_neg(exp_rate_density(rate)),
            pointwise_add(identity_fn[Real], constant[Real, Real](Real.1 / rate)), x) +
            pointwise_mul(exp_decay(rate), constant[Real, Real](Real.1), x) =
            -exp_rate_density(rate, x) * (x + Real.1 / rate) + exp_decay(rate, x)
        pointwise_add(
            pointwise_mul(pointwise_neg(exp_rate_density(rate)),
                pointwise_add(identity_fn[Real], constant[Real, Real](Real.1 / rate))),
            pointwise_mul(exp_decay(rate), constant[Real, Real](Real.1)), x) =
            -exp_rate_density(rate, x) * (x + Real.1 / rate) + exp_decay(rate, x)
        exp_rate_density(rate, x) = rate * exp_decay(rate, x)
        -(rate * exp_decay(rate, x)) * (x + Real.1 / rate) + exp_decay(rate, x) =
            exp_decay(rate, x) - rate * exp_decay(rate, x) * (x + Real.1 / rate)
        // -λe^(-λx)(x + 1/λ) + e^(-λx) = -λ·x·e^(-λx)
        // first: λ·(x + 1/λ) = λ·x + 1
        rate * (x + Real.1 / rate) = rate * x + rate * (Real.1 / rate)
        div_mul_cancel_denominator(Real.1, rate)
        (Real.1 / rate) * rate = Real.1
        real_mul_comm(Real.1 / rate, rate)
        rate * (Real.1 / rate) = Real.1
        rate * x + rate * (Real.1 / rate) = rate * x + Real.1
        rate * (x + Real.1 / rate) = rate * x + Real.1
        // then: e - λ·e·(x + 1/λ) = e·(1 - λ·(x + 1/λ)) = e·(1 - (λx + 1)) = e·(-λx)
        exp_decay(rate, x) - rate * exp_decay(rate, x) * (x + Real.1 / rate) =
            exp_decay(rate, x) - exp_decay(rate, x) * (rate * (x + Real.1 / rate))
        exp_decay(rate, x) - exp_decay(rate, x) * (rate * (x + Real.1 / rate)) =
            exp_decay(rate, x) * (Real.1 - rate * (x + Real.1 / rate))
        exp_decay(rate, x) * (Real.1 - rate * (x + Real.1 / rate)) =
            exp_decay(rate, x) * (Real.1 - (rate * x + Real.1))
        Real.1 - (rate * x + Real.1) = Real.1 - (Real.1 + rate * x)
        Real.1 - (Real.1 + rate * x) = (Real.1 - Real.1) - rate * x
        Real.1 - Real.1 = Real.0
        (Real.1 - Real.1) - rate * x = Real.0 - rate * x
        Real.0 - rate * x = -(rate * x)
        Real.1 - (rate * x + Real.1) = -(rate * x)
        exp_decay(rate, x) * (Real.1 - rate * (x + Real.1 / rate)) =
            exp_decay(rate, x) * (-(rate * x))
        exp_decay(rate, x) * (-(rate * x)) = -(exp_decay(rate, x) * (rate * x))
        exp_decay(rate, x) * (rate * x) = rate * (exp_decay(rate, x) * x)
        exp_decay(rate, x) - rate * exp_decay(rate, x) * (x + Real.1 / rate) =
            -(rate * (exp_decay(rate, x) * x))
        exp_rate_density(rate, x) = rate * exp_decay(rate, x)
        rate * (exp_decay(rate, x) * x) = x * exp_rate_density(rate, x)
        exp_decay(rate, x) - rate * exp_decay(rate, x) * (x + Real.1 / rate) =
            -(x * exp_rate_density(rate, x))
        exponential_moment_integrand(rate, x) = x * exp_rate_density(rate, x)
        pointwise_add(
            pointwise_mul(pointwise_neg(exp_rate_density(rate)),
                pointwise_add(identity_fn[Real], constant[Real, Real](Real.1 / rate))),
            pointwise_mul(exp_decay(rate), constant[Real, Real](Real.1)), x) =
            -exponential_moment_integrand(rate, x)
        pointwise_neg(exponential_moment_integrand(rate), x) =
            -exponential_moment_integrand(rate, x)
        pointwise_add(
            pointwise_mul(pointwise_neg(exp_rate_density(rate)),
                pointwise_add(identity_fn[Real], constant[Real, Real](Real.1 / rate))),
            pointwise_mul(exp_decay(rate), constant[Real, Real](Real.1)), x) =
            pointwise_neg(exponential_moment_integrand(rate), x)
    }
    function_extensionality(
        pointwise_add(
            pointwise_mul(pointwise_neg(exp_rate_density(rate)),
                pointwise_add(identity_fn[Real], constant[Real, Real](Real.1 / rate))),
            pointwise_mul(exp_decay(rate), constant[Real, Real](Real.1))),
        pointwise_neg(exponential_moment_integrand(rate)))
    derivative_fn_neg(
        pointwise_mul(exp_decay(rate),
            pointwise_add(identity_fn[Real], constant[Real, Real](Real.1 / rate))),
        pointwise_neg(exponential_moment_integrand(rate)))
    is_derivative_fn(
        pointwise_neg(pointwise_mul(exp_decay(rate),
            pointwise_add(identity_fn[Real], constant[Real, Real](Real.1 / rate)))),
        pointwise_neg(pointwise_neg(exponential_moment_integrand(rate))))
    forall(x: Real) {
        exponential_moment_anti(rate, x) =
            -exp_decay(rate, x) * (x + Real.1 / rate)
        pointwise_neg(pointwise_mul(exp_decay(rate),
            pointwise_add(identity_fn[Real], constant[Real, Real](Real.1 / rate))), x) =
            -pointwise_mul(exp_decay(rate),
                pointwise_add(identity_fn[Real], constant[Real, Real](Real.1 / rate)), x)
        pointwise_mul(exp_decay(rate),
            pointwise_add(identity_fn[Real], constant[Real, Real](Real.1 / rate)), x) =
            exp_decay(rate, x) *
                pointwise_add(identity_fn[Real], constant[Real, Real](Real.1 / rate), x)
        pointwise_add(identity_fn[Real], constant[Real, Real](Real.1 / rate), x) =
            identity_fn[Real](x) + constant[Real, Real](Real.1 / rate, x)
        identity_fn[Real](x) = x
        constant[Real, Real](Real.1 / rate, x) = Real.1 / rate
        pointwise_add(identity_fn[Real], constant[Real, Real](Real.1 / rate), x) =
            x + Real.1 / rate
        pointwise_mul(exp_decay(rate),
            pointwise_add(identity_fn[Real], constant[Real, Real](Real.1 / rate)), x) =
            exp_decay(rate, x) * (x + Real.1 / rate)
        pointwise_neg(pointwise_mul(exp_decay(rate),
            pointwise_add(identity_fn[Real], constant[Real, Real](Real.1 / rate))), x) =
            -exp_decay(rate, x) * (x + Real.1 / rate)
        exponential_moment_anti(rate, x) =
            pointwise_neg(pointwise_mul(exp_decay(rate),
                pointwise_add(identity_fn[Real], constant[Real, Real](Real.1 / rate))), x)
        pointwise_neg(pointwise_neg(exponential_moment_integrand(rate)), x) =
            -pointwise_neg(exponential_moment_integrand(rate), x)
        pointwise_neg(exponential_moment_integrand(rate), x) =
            -exponential_moment_integrand(rate, x)
        -(-exponential_moment_integrand(rate, x)) = exponential_moment_integrand(rate, x)
        pointwise_neg(pointwise_neg(exponential_moment_integrand(rate)), x) =
            exponential_moment_integrand(rate, x)
    }
    function_extensionality(
        pointwise_neg(pointwise_neg(exponential_moment_integrand(rate))),
        exponential_moment_integrand(rate))
    eq_true_intro(is_derivative_fn(
        pointwise_neg(pointwise_mul(exp_decay(rate),
            pointwise_add(identity_fn[Real], constant[Real, Real](Real.1 / rate)))),
        pointwise_neg(pointwise_neg(exponential_moment_integrand(rate)))))
    (is_derivative_fn(
        pointwise_neg(pointwise_mul(exp_decay(rate),
            pointwise_add(identity_fn[Real], constant[Real, Real](Real.1 / rate)))),
        pointwise_neg(pointwise_neg(exponential_moment_integrand(rate))))) = true
    function_eq_transport_predicate_rev[Real, Real](
        function(h: Real -> Real) {
            is_derivative_fn(
                pointwise_neg(pointwise_mul(exp_decay(rate),
                    pointwise_add(identity_fn[Real], constant[Real, Real](Real.1 / rate)))), h)
        },
        pointwise_neg(pointwise_neg(exponential_moment_integrand(rate))),
        exponential_moment_integrand(rate))
    is_derivative_fn(
        pointwise_neg(pointwise_mul(exp_decay(rate),
            pointwise_add(identity_fn[Real], constant[Real, Real](Real.1 / rate)))),
        exponential_moment_integrand(rate))
    is_derivative_fn(
        pointwise_neg(pointwise_mul(exp_decay(rate),
            pointwise_add(identity_fn[Real], constant[Real, Real](Real.1 / rate)))),
        exponential_moment_integrand(rate))
    }
}

/// The moment integrand has derivative t ↦ λ·e^(-λt)·(1 - λ·t).
theorem exponential_moment_derivative(rate: Real) {
    is_derivative_fn(exponential_moment_integrand(rate), exponential_moment_deriv(rate))
} by {
    exp_rate_density_derivative(rate)
    is_derivative_fn(exp_rate_density(rate), exp_rate_density_deriv(rate))
    derivative_fn_identity
    is_derivative_fn(identity_fn[Real], constant[Real, Real](Real.1))
    derivative_fn_mul(identity_fn[Real], exp_rate_density(rate),
        constant[Real, Real](Real.1), exp_rate_density_deriv(rate))
    is_derivative_fn(
        pointwise_mul(identity_fn[Real], exp_rate_density(rate)),
        pointwise_add(
            pointwise_mul(identity_fn[Real], exp_rate_density_deriv(rate)),
            pointwise_mul(exp_rate_density(rate), constant[Real, Real](Real.1))))
    forall(x: Real) {
        exponential_moment_integrand(rate, x) = x * exp_rate_density(rate, x)
        pointwise_mul(identity_fn[Real], exp_rate_density(rate), x) =
            identity_fn[Real](x) * exp_rate_density(rate, x)
        identity_fn[Real](x) = x
        pointwise_mul(identity_fn[Real], exp_rate_density(rate), x) =
            x * exp_rate_density(rate, x)
        exponential_moment_integrand(rate, x) =
            pointwise_mul(identity_fn[Real], exp_rate_density(rate), x)
        pointwise_mul(identity_fn[Real], exp_rate_density_deriv(rate), x) =
            identity_fn[Real](x) * exp_rate_density_deriv(rate, x)
        identity_fn[Real](x) = x
        pointwise_mul(identity_fn[Real], exp_rate_density_deriv(rate), x) =
            x * exp_rate_density_deriv(rate, x)
        exp_rate_density_deriv(rate, x) = -(rate * rate) * exp_decay(rate, x)
        x * exp_rate_density_deriv(rate, x) = x * (-(rate * rate) * exp_decay(rate, x))
        x * (-(rate * rate) * exp_decay(rate, x)) = -(x * ((rate * rate) * exp_decay(rate, x)))
        x * ((rate * rate) * exp_decay(rate, x)) = (rate * rate) * (x * exp_decay(rate, x))
        // keep it symbolic: rate · (x·e^(-λx)) = x·(rate·e^(-λx))
        real_mul_comm(rate, exp_decay(rate, x))
        rate * exp_decay(rate, x) = exp_decay(rate, x) * rate
        exp_rate_density(rate, x) = rate * exp_decay(rate, x)
        (rate * rate) * (x * exp_decay(rate, x)) = rate * (x * exp_rate_density(rate, x))
        x * ((rate * rate) * exp_decay(rate, x)) = rate * (x * exp_rate_density(rate, x))
        x * exp_rate_density(rate, x) = exponential_moment_integrand(rate, x)
        x * ((rate * rate) * exp_decay(rate, x)) = rate * exponential_moment_integrand(rate, x)
        x * (-(rate * rate) * exp_decay(rate, x)) =
            -(rate * exponential_moment_integrand(rate, x))
        pointwise_mul(identity_fn[Real], exp_rate_density_deriv(rate), x) =
            -(rate * exponential_moment_integrand(rate, x))
        pointwise_mul(exp_rate_density(rate), constant[Real, Real](Real.1), x) =
            exp_rate_density(rate, x) * constant[Real, Real](Real.1, x)
        constant[Real, Real](Real.1, x) = Real.1
        pointwise_mul(exp_rate_density(rate), constant[Real, Real](Real.1), x) =
            exp_rate_density(rate, x)
        pointwise_add(
            pointwise_mul(identity_fn[Real], exp_rate_density_deriv(rate)),
            pointwise_mul(exp_rate_density(rate), constant[Real, Real](Real.1)), x) =
            pointwise_mul(identity_fn[Real], exp_rate_density_deriv(rate), x) +
            pointwise_mul(exp_rate_density(rate), constant[Real, Real](Real.1), x)
        pointwise_mul(identity_fn[Real], exp_rate_density_deriv(rate), x) +
            pointwise_mul(exp_rate_density(rate), constant[Real, Real](Real.1), x) =
            -(rate * exponential_moment_integrand(rate, x)) + exp_rate_density(rate, x)
        pointwise_add(
            pointwise_mul(identity_fn[Real], exp_rate_density_deriv(rate)),
            pointwise_mul(exp_rate_density(rate), constant[Real, Real](Real.1)), x) =
            -(rate * exponential_moment_integrand(rate, x)) + exp_rate_density(rate, x)
        exp_rate_density(rate, x) - rate * exponential_moment_integrand(rate, x) =
            exp_rate_density(rate, x) * (Real.1 - rate * x)
        exponential_moment_deriv(rate, x) =
            exp_rate_density(rate, x) * (Real.1 - rate * x)
        pointwise_add(
            pointwise_mul(identity_fn[Real], exp_rate_density_deriv(rate)),
            pointwise_mul(exp_rate_density(rate), constant[Real, Real](Real.1)), x) =
            exponential_moment_deriv(rate, x)
    }
    function_extensionality(
        pointwise_mul(identity_fn[Real], exp_rate_density(rate)),
        exponential_moment_integrand(rate))
    function_extensionality(
        pointwise_add(
            pointwise_mul(identity_fn[Real], exp_rate_density_deriv(rate)),
            pointwise_mul(exp_rate_density(rate), constant[Real, Real](Real.1))),
        exponential_moment_deriv(rate))
    define moment_deriv_pred(h: Real -> Real) -> Bool {
        is_derivative_fn(pointwise_mul(identity_fn[Real], exp_rate_density(rate)), h)
    }
    moment_deriv_pred(pointwise_add(
        pointwise_mul(identity_fn[Real], exp_rate_density_deriv(rate)),
        pointwise_mul(exp_rate_density(rate), constant[Real, Real](Real.1))))
    function_eq_transport_predicate_rev(moment_deriv_pred, exponential_moment_deriv(rate),
        pointwise_add(
            pointwise_mul(identity_fn[Real], exp_rate_density_deriv(rate)),
            pointwise_mul(exp_rate_density(rate), constant[Real, Real](Real.1))))
    is_derivative_fn(pointwise_mul(identity_fn[Real], exp_rate_density(rate)),
        exponential_moment_deriv(rate))
    eq_true_intro(is_derivative_fn(pointwise_mul(identity_fn[Real], exp_rate_density(rate)),
        exponential_moment_deriv(rate)))
    (is_derivative_fn(pointwise_mul(identity_fn[Real], exp_rate_density(rate)),
        exponential_moment_deriv(rate))) = true
    function_eq_transport_predicate_rev[Real, Real](
        function(h: Real -> Real) { is_derivative_fn(h, exponential_moment_deriv(rate)) },
        exponential_moment_integrand(rate),
        pointwise_mul(identity_fn[Real], exp_rate_density(rate)))
    is_derivative_fn(exponential_moment_integrand(rate), exponential_moment_deriv(rate))
}

/// The moment integrand is continuous.
theorem exponential_moment_continuous(rate: Real) {
    continuous(exponential_moment_integrand(rate))
} by {
    identity_function_is_continuous
    continuous(identity_fn[Real])
    exp_rate_density_continuous(rate)
    continuous(exp_rate_density(rate))
    continuous_pointwise_mul(identity_fn[Real], exp_rate_density(rate))
    continuous(pointwise_mul(identity_fn[Real], exp_rate_density(rate)))
    forall(x: Real) {
        exponential_moment_integrand(rate, x) = x * exp_rate_density(rate, x)
        pointwise_mul(identity_fn[Real], exp_rate_density(rate), x) =
            identity_fn[Real](x) * exp_rate_density(rate, x)
        identity_fn[Real](x) = x
        pointwise_mul(identity_fn[Real], exp_rate_density(rate), x) =
            x * exp_rate_density(rate, x)
        exponential_moment_integrand(rate, x) =
            pointwise_mul(identity_fn[Real], exp_rate_density(rate), x)
    }
    function_extensionality(exponential_moment_integrand(rate),
        pointwise_mul(identity_fn[Real], exp_rate_density(rate)))
    define moment_cont_pred(h: Real -> Real) -> Bool {
        continuous(h)
    }
    moment_cont_pred(pointwise_mul(identity_fn[Real], exp_rate_density(rate)))
    function_eq_transport_predicate_rev(moment_cont_pred, exponential_moment_integrand(rate),
        pointwise_mul(identity_fn[Real], exp_rate_density(rate)))
    continuous(exponential_moment_integrand(rate))
}

/// The derivative of the moment integrand is bounded on [0, b].
theorem exponential_moment_deriv_bound(rate: Real, b: Real, z: Real) {
    rate > Real.0 and Real.0 <= b and interval_contains(Real.0, b, z)
    implies exponential_moment_deriv(rate, z).abs <= rate * (Real.1 + rate * b)
} by {
    if rate > Real.0 and Real.0 <= b and interval_contains(Real.0, b, z) {
        // rate > 0, 1 + rate*b > 0 → product ≥ 0
        lt_imp_lte(Real.0, rate)
        Real.0 <= rate
        interval_contains_left(Real.0, b, z)
        Real.0 <= z
        exponential_moment_deriv(rate, z) =
            exp_rate_density(rate, z) * (Real.1 - rate * z)
                // |λe^(-λz)(1 - λz)| = λe^(-λz)·|1 - λz|
                mul_abs(exp_rate_density(rate, z), Real.1 - rate * z)
                (exp_rate_density(rate, z) * (Real.1 - rate * z)).abs =
                    exp_rate_density(rate, z).abs * (Real.1 - rate * z).abs
                exponential_moment_deriv(rate, z).abs =
                    exp_rate_density(rate, z).abs * (Real.1 - rate * z).abs
                exp_rate_density_nonneg(rate, z)
                Real.0 <= exp_rate_density(rate, z)
                abs_of_nonneg(exp_rate_density(rate, z))
                exp_rate_density(rate, z).abs = exp_rate_density(rate, z)
                exponential_moment_deriv(rate, z).abs =
                    exp_rate_density(rate, z) * (Real.1 - rate * z).abs
                // |1 - λz| <= 1 + λz <= 1 + λb
                triangle_ineq(Real.1, -(rate * z))
                (Real.1 + -(rate * z)).abs <= Real.1.abs + (-(rate * z)).abs
                Real.1 - rate * z = Real.1 + -(rate * z)
                (Real.1 - rate * z).abs <= Real.1.abs + (-(rate * z)).abs
                Real.1.abs = Real.1
                abs_neg(rate * z)
                (-(rate * z)).abs = (rate * z).abs
                lt_imp_lte(Real.0, rate)
                Real.0 <= rate
                mul_nonneg_lte(rate, z)
                Real.0 <= rate * z
                abs_of_nonneg(rate * z)
                (rate * z).abs = rate * z
                (-(rate * z)).abs = rate * z
                Real.1.abs + (-(rate * z)).abs = Real.1 + rate * z
                (Real.1 - rate * z).abs <= Real.1 + rate * z
                interval_contains_right(Real.0, b, z)
                z <= b
                Real.0 <= rate
                mul_le_mul_of_nonneg_right(z, b, rate)
                z * rate <= b * rate
                real_mul_comm(rate, z)
                rate * z = z * rate
                real_mul_comm(rate, b)
                rate * b = b * rate
                rate * z <= rate * b
                add_le_add_right(Real.1, Real.1, rate * z)
                Real.1 + rate * z <= Real.1 + rate * b
                lte_trans((Real.1 - rate * z).abs, Real.1 + rate * z, Real.1 + rate * b)
                (Real.1 - rate * z).abs <= Real.1 + rate * b
                // λe^(-λz) ≤ λ and 0 ≤ |1-λz|
                exp_rate_density_le_rate(rate, z)
                exp_rate_density(rate, z) <= rate
                abs_gte_zero(Real.1 - rate * z)
                Real.0 <= (Real.1 - rate * z).abs
                exp_rate_density_nonneg(rate, z)
                Real.0 <= exp_rate_density(rate, z)
                Real.0 <= rate
                lt_imp_lte(Real.0, rate)
                Real.0 <= z
                mul_nonneg_lte(rate, z)
                Real.0 <= rate * z
                Real.0 <= b
                mul_nonneg_lte(rate, b)
                Real.0 <= rate * b
                zero_is_smaller_than_one[Real]
                Real.0 < Real.1
                lt_imp_lte(Real.0, Real.1)
                Real.0 <= Real.1
                add_le_add(Real.0, Real.1, Real.0, rate * b)
                Real.0 + Real.0 <= Real.1 + rate * b
                Real.0 + Real.0 = Real.0
                Real.0 <= Real.1 + rate * b
                mul_le_mul_nonneg_lte(exp_rate_density(rate, z), (Real.1 - rate * z).abs,
                    rate, Real.1 + rate * b)
                exp_rate_density(rate, z) * (Real.1 - rate * z).abs <= rate * (Real.1 + rate * b)
                exponential_moment_deriv(rate, z).abs <= rate * (Real.1 + rate * b)
    }
}

/// The moment integrand is bounded below by zero on [0, b].
theorem exponential_moment_lower_bound_on(rate: Real, b: Real, t: Real) {
    rate > Real.0 and Real.0 <= b and interval_contains(Real.0, b, t)
    implies Real.0 <= exponential_moment_integrand(rate, t)
} by {
    if rate > Real.0 and Real.0 <= b and interval_contains(Real.0, b, t) {
        exponential_moment_integrand(rate, t) = t * exp_rate_density(rate, t)
        interval_contains_left(Real.0, b, t)
        Real.0 <= t
        exp_rate_density_nonneg(rate, t)
        Real.0 <= exp_rate_density(rate, t)
        mul_nonneg_lte(t, exp_rate_density(rate, t))
        Real.0 <= t * exp_rate_density(rate, t)
        Real.0 <= exponential_moment_integrand(rate, t)
    }
}

/// The moment integrand is bounded above by b·λ on [0, b].
theorem exponential_moment_upper_bound_on(rate: Real, b: Real, t: Real) {
    rate > Real.0 and Real.0 <= b and interval_contains(Real.0, b, t)
    implies exponential_moment_integrand(rate, t) <= b * rate
} by {
    if rate > Real.0 and Real.0 <= b and interval_contains(Real.0, b, t) {
        exponential_moment_integrand(rate, t) = t * exp_rate_density(rate, t)
        interval_contains_left(Real.0, b, t)
        Real.0 <= t
        exp_rate_density_nonneg(rate, t)
        Real.0 <= exp_rate_density(rate, t)
        interval_contains_right(Real.0, b, t)
        t <= b
        lt_imp_lte(Real.0, rate)
        Real.0 <= rate
        exp_rate_density_le_rate(rate, t)
        exp_rate_density(rate, t) <= rate
        Real.0 <= t
        Real.0 <= b
        Real.0 <= exp_rate_density(rate, t)
        Real.0 <= rate
        mul_le_mul_nonneg_lte(t, exp_rate_density(rate, t), b, rate)
        t * exp_rate_density(rate, t) <= b * rate
        exponential_moment_integrand(rate, t) <= b * rate
    }
}

/// The moment integrand is integrable on [0, b].
theorem exponential_moment_integrable(rate: Real, b: Real) {
    rate > Real.0 and Real.0 <= b implies
    is_integrable(exponential_moment_integrand(rate), Real.0, b)
} by {
    if rate > Real.0 and Real.0 <= b {
        forall(z: Real) {
            if interval_contains(Real.0, b, z) {
                exponential_moment_deriv_bound(rate, b, z)
                exponential_moment_deriv(rate, z).abs <= rate * (Real.1 + rate * b)
            }
        }
        forall(t: Real) {
            if interval_contains(Real.0, b, t) {
                exponential_moment_lower_bound_on(rate, b, t)
                Real.0 <= exponential_moment_integrand(rate, t)
            }
        }
        forall(t: Real) {
            if interval_contains(Real.0, b, t) {
                exponential_moment_upper_bound_on(rate, b, t)
                exponential_moment_integrand(rate, t) <= b * rate
            }
        }
        // 0 <= rate * (1 + rate*b)
        lt_imp_lte(Real.0, rate)
        Real.0 <= rate
        zero_is_smaller_than_one[Real]
        Real.0 < Real.1
        lt_imp_lte(Real.0, Real.1)
        Real.0 <= Real.1
        mul_nonneg_lte(rate, b)
        Real.0 <= rate * b
        add_le_add(Real.0, Real.1, Real.0, rate * b)
        Real.0 + Real.0 <= Real.1 + rate * b
        Real.0 + Real.0 = Real.0
        Real.0 <= Real.1 + rate * b
        mul_nonneg_lte(rate, Real.1 + rate * b)
        Real.0 <= rate * (Real.1 + rate * b)
        exponential_moment_anti_derivative(rate)
        is_derivative_fn(pointwise_neg(pointwise_mul(exp_decay(rate),
        pointwise_add(identity_fn[Real], constant[Real, Real](Real.1 / rate)))),
        exponential_moment_integrand(rate))
        exponential_moment_derivative(rate)
        is_derivative_fn(exponential_moment_integrand(rate), exponential_moment_deriv(rate))
        exponential_moment_continuous(rate)
        continuous(exponential_moment_integrand(rate))
        // continuity of the antiderivative
        exp_decay_continuous(rate)
        continuous(exp_decay(rate))
        continuous_pointwise_add(identity_fn[Real], constant[Real, Real](Real.1 / rate))
        continuous(pointwise_add(identity_fn[Real], constant[Real, Real](Real.1 / rate)))
        continuous_pointwise_mul(exp_decay(rate),
            pointwise_add(identity_fn[Real], constant[Real, Real](Real.1 / rate)))
        continuous(pointwise_mul(exp_decay(rate),
            pointwise_add(identity_fn[Real], constant[Real, Real](Real.1 / rate))))
        continuous_pointwise_neg(pointwise_mul(exp_decay(rate),
            pointwise_add(identity_fn[Real], constant[Real, Real](Real.1 / rate))))
        continuous(pointwise_neg(pointwise_mul(exp_decay(rate),
            pointwise_add(identity_fn[Real], constant[Real, Real](Real.1 / rate)))))
        forall(x: Real) {
            exponential_moment_anti(rate, x) =
                -exp_decay(rate, x) * (x + Real.1 / rate)
            pointwise_neg(pointwise_mul(exp_decay(rate),
                pointwise_add(identity_fn[Real], constant[Real, Real](Real.1 / rate))), x) =
                -pointwise_mul(exp_decay(rate),
                    pointwise_add(identity_fn[Real], constant[Real, Real](Real.1 / rate)), x)
            pointwise_mul(exp_decay(rate),
                pointwise_add(identity_fn[Real], constant[Real, Real](Real.1 / rate)), x) =
                exp_decay(rate, x) *
                    pointwise_add(identity_fn[Real], constant[Real, Real](Real.1 / rate), x)
            pointwise_add(identity_fn[Real], constant[Real, Real](Real.1 / rate), x) =
                identity_fn[Real](x) + constant[Real, Real](Real.1 / rate, x)
            identity_fn[Real](x) = x
            constant[Real, Real](Real.1 / rate, x) = Real.1 / rate
            pointwise_add(identity_fn[Real], constant[Real, Real](Real.1 / rate), x) =
                x + Real.1 / rate
            pointwise_mul(exp_decay(rate),
                pointwise_add(identity_fn[Real], constant[Real, Real](Real.1 / rate)), x) =
                exp_decay(rate, x) * (x + Real.1 / rate)
            pointwise_neg(pointwise_mul(exp_decay(rate),
                pointwise_add(identity_fn[Real], constant[Real, Real](Real.1 / rate))), x) =
                -exp_decay(rate, x) * (x + Real.1 / rate)
            exponential_moment_anti(rate, x) =
                pointwise_neg(pointwise_mul(exp_decay(rate),
                    pointwise_add(identity_fn[Real], constant[Real, Real](Real.1 / rate))), x)
        }
        continuous(pointwise_neg(pointwise_mul(exp_decay(rate),
        pointwise_add(identity_fn[Real], constant[Real, Real](Real.1 / rate)))))
        eq_true_intro(forall(z: Real) {
            interval_contains(Real.0, b, z) implies
            exponential_moment_deriv(rate, z).abs <= rate * (Real.1 + rate * b)
        })
        (forall(z: Real) {
            interval_contains(Real.0, b, z) implies
            exponential_moment_deriv(rate, z).abs <= rate * (Real.1 + rate * b)
        }) = true
        eq_true_intro(forall(t: Real) {
            interval_contains(Real.0, b, t) implies Real.0 <= exponential_moment_integrand(rate, t)
        })
        (forall(t: Real) {
            interval_contains(Real.0, b, t) implies Real.0 <= exponential_moment_integrand(rate, t)
        }) = true
        eq_true_intro(forall(t: Real) {
            interval_contains(Real.0, b, t) implies exponential_moment_integrand(rate, t) <= b * rate
        })
        (forall(t: Real) {
            interval_contains(Real.0, b, t) implies exponential_moment_integrand(rate, t) <= b * rate
        }) = true
        eq_true_intro(continuous(pointwise_neg(pointwise_mul(exp_decay(rate),
        pointwise_add(identity_fn[Real], constant[Real, Real](Real.1 / rate))))))
        (continuous(pointwise_neg(pointwise_mul(exp_decay(rate),
        pointwise_add(identity_fn[Real], constant[Real, Real](Real.1 / rate)))))) = true
        eq_true_intro(is_derivative_fn(pointwise_neg(pointwise_mul(exp_decay(rate),
        pointwise_add(identity_fn[Real], constant[Real, Real](Real.1 / rate)))), exponential_moment_integrand(rate)))
        (is_derivative_fn(pointwise_neg(pointwise_mul(exp_decay(rate),
        pointwise_add(identity_fn[Real], constant[Real, Real](Real.1 / rate)))), exponential_moment_integrand(rate))) = true
        eq_true_intro(is_derivative_fn(exponential_moment_integrand(rate), exponential_moment_deriv(rate)))
        (is_derivative_fn(exponential_moment_integrand(rate), exponential_moment_deriv(rate))) = true
        eq_true_intro(continuous(exponential_moment_integrand(rate)))
        (continuous(exponential_moment_integrand(rate))) = true
        Real.0 <= b and Real.0 <= rate * (Real.1 + rate * b) and
            continuous(pointwise_neg(pointwise_mul(exp_decay(rate),
        pointwise_add(identity_fn[Real], constant[Real, Real](Real.1 / rate))))) and
            is_derivative_fn(pointwise_neg(pointwise_mul(exp_decay(rate),
        pointwise_add(identity_fn[Real], constant[Real, Real](Real.1 / rate)))), exponential_moment_integrand(rate)) and
            is_derivative_fn(exponential_moment_integrand(rate), exponential_moment_deriv(rate)) and
            continuous(exponential_moment_integrand(rate)) and
            (forall(z: Real) {
                interval_contains(Real.0, b, z) implies
                exponential_moment_deriv(rate, z).abs <= rate * (Real.1 + rate * b)
            }) and
            (forall(t: Real) {
                interval_contains(Real.0, b, t) implies Real.0 <= exponential_moment_integrand(rate, t)
            }) and
            (forall(t: Real) {
                interval_contains(Real.0, b, t) implies exponential_moment_integrand(rate, t) <= b * rate
            })
        fn_integrable_gen_derivative_bound(
            exponential_moment_integrand(rate), pointwise_neg(pointwise_mul(exp_decay(rate),
        pointwise_add(identity_fn[Real], constant[Real, Real](Real.1 / rate)))),
            exponential_moment_deriv(rate), Real.0, b,
            rate * (Real.1 + rate * b), Real.0, b * rate)
        is_integrable(exponential_moment_integrand(rate), Real.0, b)
    }
}

/// The expectation of X under the exponential law over [0, b] is
/// 1/λ - e^(-λb)·(b + 1/λ), which tends to 1/λ as b grows.
theorem exponential_mean_interval(rate: Real, b: Real) {
    rate > Real.0 and Real.0 <= b implies
    integral(exponential_moment_integrand(rate), Real.0, b) =
        Real.1 / rate - exp_decay(rate, b) * (b + Real.1 / rate)
} by {
    if rate > Real.0 and Real.0 <= b {
        exponential_moment_integrable(rate, b)
        is_integrable(exponential_moment_integrand(rate), Real.0, b)
        forall(z: Real) {
            if interval_contains(Real.0, b, z) {
                exponential_moment_deriv_bound(rate, b, z)
                exponential_moment_deriv(rate, z).abs <= rate * (Real.1 + rate * b)
            }
        }
        forall(t: Real) {
            if interval_contains(Real.0, b, t) {
                exponential_moment_lower_bound_on(rate, b, t)
                Real.0 <= exponential_moment_integrand(rate, t)
            }
        }
        forall(t: Real) {
            if interval_contains(Real.0, b, t) {
                exponential_moment_upper_bound_on(rate, b, t)
                exponential_moment_integrand(rate, t) <= b * rate
            }
        }
        lt_imp_lte(Real.0, rate)
        Real.0 <= rate
        zero_is_smaller_than_one[Real]
        Real.0 < Real.1
        lt_imp_lte(Real.0, Real.1)
        Real.0 <= Real.1
        mul_nonneg_lte(rate, b)
        Real.0 <= rate * b
        add_le_add(Real.0, Real.1, Real.0, rate * b)
        Real.0 + Real.0 <= Real.1 + rate * b
        Real.0 + Real.0 = Real.0
        Real.0 <= Real.1 + rate * b
        mul_nonneg_lte(rate, Real.1 + rate * b)
        Real.0 <= rate * (Real.1 + rate * b)
        exponential_moment_anti_derivative(rate)
        is_derivative_fn(pointwise_neg(pointwise_mul(exp_decay(rate),
        pointwise_add(identity_fn[Real], constant[Real, Real](Real.1 / rate)))),
        exponential_moment_integrand(rate))
        exponential_moment_derivative(rate)
        is_derivative_fn(exponential_moment_integrand(rate), exponential_moment_deriv(rate))
        exponential_moment_continuous(rate)
        continuous(exponential_moment_integrand(rate))
        exp_decay_continuous(rate)
        continuous(exp_decay(rate))
        continuous_pointwise_add(identity_fn[Real], constant[Real, Real](Real.1 / rate))
        continuous(pointwise_add(identity_fn[Real], constant[Real, Real](Real.1 / rate)))
        continuous_pointwise_mul(exp_decay(rate),
            pointwise_add(identity_fn[Real], constant[Real, Real](Real.1 / rate)))
        continuous(pointwise_mul(exp_decay(rate),
            pointwise_add(identity_fn[Real], constant[Real, Real](Real.1 / rate))))
        continuous_pointwise_neg(pointwise_mul(exp_decay(rate),
            pointwise_add(identity_fn[Real], constant[Real, Real](Real.1 / rate))))
        continuous(pointwise_neg(pointwise_mul(exp_decay(rate),
            pointwise_add(identity_fn[Real], constant[Real, Real](Real.1 / rate)))))
        forall(x: Real) {
            exponential_moment_anti(rate, x) =
                -exp_decay(rate, x) * (x + Real.1 / rate)
            pointwise_neg(pointwise_mul(exp_decay(rate),
                pointwise_add(identity_fn[Real], constant[Real, Real](Real.1 / rate))), x) =
                -pointwise_mul(exp_decay(rate),
                    pointwise_add(identity_fn[Real], constant[Real, Real](Real.1 / rate)), x)
            pointwise_mul(exp_decay(rate),
                pointwise_add(identity_fn[Real], constant[Real, Real](Real.1 / rate)), x) =
                exp_decay(rate, x) *
                    pointwise_add(identity_fn[Real], constant[Real, Real](Real.1 / rate), x)
            pointwise_add(identity_fn[Real], constant[Real, Real](Real.1 / rate), x) =
                identity_fn[Real](x) + constant[Real, Real](Real.1 / rate, x)
            identity_fn[Real](x) = x
            constant[Real, Real](Real.1 / rate, x) = Real.1 / rate
            pointwise_add(identity_fn[Real], constant[Real, Real](Real.1 / rate), x) =
                x + Real.1 / rate
            pointwise_mul(exp_decay(rate),
                pointwise_add(identity_fn[Real], constant[Real, Real](Real.1 / rate)), x) =
                exp_decay(rate, x) * (x + Real.1 / rate)
            pointwise_neg(pointwise_mul(exp_decay(rate),
                pointwise_add(identity_fn[Real], constant[Real, Real](Real.1 / rate))), x) =
                -exp_decay(rate, x) * (x + Real.1 / rate)
            exponential_moment_anti(rate, x) =
                pointwise_neg(pointwise_mul(exp_decay(rate),
                    pointwise_add(identity_fn[Real], constant[Real, Real](Real.1 / rate))), x)
        }
        continuous(pointwise_neg(pointwise_mul(exp_decay(rate),
        pointwise_add(identity_fn[Real], constant[Real, Real](Real.1 / rate)))))
        eq_true_intro(forall(z: Real) {
            interval_contains(Real.0, b, z) implies
            exponential_moment_deriv(rate, z).abs <= rate * (Real.1 + rate * b)
        })
        (forall(z: Real) {
            interval_contains(Real.0, b, z) implies
            exponential_moment_deriv(rate, z).abs <= rate * (Real.1 + rate * b)
        }) = true
        eq_true_intro(forall(t: Real) {
            interval_contains(Real.0, b, t) implies Real.0 <= exponential_moment_integrand(rate, t)
        })
        (forall(t: Real) {
            interval_contains(Real.0, b, t) implies Real.0 <= exponential_moment_integrand(rate, t)
        }) = true
        eq_true_intro(forall(t: Real) {
            interval_contains(Real.0, b, t) implies exponential_moment_integrand(rate, t) <= b * rate
        })
        (forall(t: Real) {
            interval_contains(Real.0, b, t) implies exponential_moment_integrand(rate, t) <= b * rate
        }) = true
        eq_true_intro(continuous(pointwise_neg(pointwise_mul(exp_decay(rate),
        pointwise_add(identity_fn[Real], constant[Real, Real](Real.1 / rate))))))
        (continuous(pointwise_neg(pointwise_mul(exp_decay(rate),
        pointwise_add(identity_fn[Real], constant[Real, Real](Real.1 / rate)))))) = true
        eq_true_intro(is_derivative_fn(pointwise_neg(pointwise_mul(exp_decay(rate),
        pointwise_add(identity_fn[Real], constant[Real, Real](Real.1 / rate)))), exponential_moment_integrand(rate)))
        (is_derivative_fn(pointwise_neg(pointwise_mul(exp_decay(rate),
        pointwise_add(identity_fn[Real], constant[Real, Real](Real.1 / rate)))), exponential_moment_integrand(rate))) = true
        eq_true_intro(is_derivative_fn(exponential_moment_integrand(rate), exponential_moment_deriv(rate)))
        (is_derivative_fn(exponential_moment_integrand(rate), exponential_moment_deriv(rate))) = true
        eq_true_intro(continuous(exponential_moment_integrand(rate)))
        (continuous(exponential_moment_integrand(rate))) = true
        Real.0 <= b and Real.0 <= rate * (Real.1 + rate * b) and
            continuous(pointwise_neg(pointwise_mul(exp_decay(rate),
        pointwise_add(identity_fn[Real], constant[Real, Real](Real.1 / rate))))) and
            is_derivative_fn(pointwise_neg(pointwise_mul(exp_decay(rate),
        pointwise_add(identity_fn[Real], constant[Real, Real](Real.1 / rate)))), exponential_moment_integrand(rate)) and
            is_derivative_fn(exponential_moment_integrand(rate), exponential_moment_deriv(rate)) and
            continuous(exponential_moment_integrand(rate)) and
            (forall(z: Real) {
                interval_contains(Real.0, b, z) implies
                exponential_moment_deriv(rate, z).abs <= rate * (Real.1 + rate * b)
            }) and
            (forall(t: Real) {
                interval_contains(Real.0, b, t) implies Real.0 <= exponential_moment_integrand(rate, t)
            }) and
            (forall(t: Real) {
                interval_contains(Real.0, b, t) implies exponential_moment_integrand(rate, t) <= b * rate
            })
        integral_ftc2_derivative_bound(
            exponential_moment_integrand(rate), pointwise_neg(pointwise_mul(exp_decay(rate),
        pointwise_add(identity_fn[Real], constant[Real, Real](Real.1 / rate)))),
            exponential_moment_deriv(rate), Real.0, b,
            rate * (Real.1 + rate * b), Real.0, b * rate)
        is_integrable(exponential_moment_integrand(rate), Real.0, b) and
            integral(exponential_moment_integrand(rate), Real.0, b) =
                pointwise_neg(pointwise_mul(exp_decay(rate),
        pointwise_add(identity_fn[Real], constant[Real, Real](Real.1 / rate))))(b) - pointwise_neg(pointwise_mul(exp_decay(rate),
        pointwise_add(identity_fn[Real], constant[Real, Real](Real.1 / rate))))(Real.0)
        integral(exponential_moment_integrand(rate), Real.0, b) =
            pointwise_neg(pointwise_mul(exp_decay(rate),
        pointwise_add(identity_fn[Real], constant[Real, Real](Real.1 / rate))))(b) - pointwise_neg(pointwise_mul(exp_decay(rate),
        pointwise_add(identity_fn[Real], constant[Real, Real](Real.1 / rate))))(Real.0)
        pointwise_neg(pointwise_mul(exp_decay(rate),
        pointwise_add(identity_fn[Real], constant[Real, Real](Real.1 / rate))))(b) =
            -exp_decay(rate, b) * (b + Real.1 / rate)
        pointwise_neg(pointwise_mul(exp_decay(rate),
            pointwise_add(identity_fn[Real], constant[Real, Real](Real.1 / rate))), Real.0) =
            -pointwise_mul(exp_decay(rate),
                pointwise_add(identity_fn[Real], constant[Real, Real](Real.1 / rate)), Real.0)
        pointwise_mul(exp_decay(rate),
            pointwise_add(identity_fn[Real], constant[Real, Real](Real.1 / rate)), Real.0) =
            exp_decay(rate, Real.0) *
                pointwise_add(identity_fn[Real], constant[Real, Real](Real.1 / rate), Real.0)
        pointwise_add(identity_fn[Real], constant[Real, Real](Real.1 / rate), Real.0) =
            identity_fn[Real](Real.0) + constant[Real, Real](Real.1 / rate, Real.0)
        identity_fn[Real](Real.0) = Real.0
        constant[Real, Real](Real.1 / rate, Real.0) = Real.1 / rate
        pointwise_add(identity_fn[Real], constant[Real, Real](Real.1 / rate), Real.0) =
            Real.0 + Real.1 / rate
        rate * Real.0 = Real.0
        -(rate * Real.0) = Real.0
        exp_decay(rate, Real.0) = (-(rate * Real.0)).exp
        exp_decay(rate, Real.0) = (Real.0).exp
        exp_zero
        (Real.0).exp = Real.1
        exp_decay(rate, Real.0) = Real.1
        Real.0 + Real.1 / rate = Real.1 / rate
        pointwise_add(identity_fn[Real], constant[Real, Real](Real.1 / rate), Real.0) =
            Real.1 / rate
        pointwise_mul(exp_decay(rate),
            pointwise_add(identity_fn[Real], constant[Real, Real](Real.1 / rate)), Real.0) =
            Real.1 * (Real.1 / rate)
        Real.1 * (Real.1 / rate) = Real.1 / rate
        pointwise_mul(exp_decay(rate),
            pointwise_add(identity_fn[Real], constant[Real, Real](Real.1 / rate)), Real.0) =
            Real.1 / rate
        pointwise_neg(pointwise_mul(exp_decay(rate),
            pointwise_add(identity_fn[Real], constant[Real, Real](Real.1 / rate))), Real.0) =
            -(Real.1 / rate)
        pointwise_neg(pointwise_mul(exp_decay(rate),
            pointwise_add(identity_fn[Real], constant[Real, Real](Real.1 / rate))))(b) -
            pointwise_neg(pointwise_mul(exp_decay(rate),
                pointwise_add(identity_fn[Real], constant[Real, Real](Real.1 / rate))))(Real.0) =
            -exp_decay(rate, b) * (b + Real.1 / rate) - (-(Real.1 / rate))
        -exp_decay(rate, b) * (b + Real.1 / rate) - (-(Real.1 / rate)) =
            -exp_decay(rate, b) * (b + Real.1 / rate) + Real.1 / rate
        -exp_decay(rate, b) * (b + Real.1 / rate) + Real.1 / rate =
            Real.1 / rate - exp_decay(rate, b) * (b + Real.1 / rate)
        integral(exponential_moment_integrand(rate), Real.0, b) =
            Real.1 / rate - exp_decay(rate, b) * (b + Real.1 / rate)
    }
}

// ---------------------------------------------------------------------------
// Limits as b → ∞ (documented; the library has no improper integrals)
// ---------------------------------------------------------------------------
//
// The finite-interval results above tend to the classical values as b → ∞:
//
//   - the total mass:  integral(f, 0, b) = 1 - e^(-λb) → 1,
//   - the expectation: integral(t·f(t), 0, b) = 1/λ - e^(-λb)·(b + 1/λ) → 1/λ,
//   - the variance:    integral((t - 1/λ)²·f(t), 0, b) → 1/λ²,
//
// because e^(-λb) → 0 and e^(-λb)·(b + 1/λ) → 0 as b → ∞ (the polynomial
// factor is dominated by the exponential decay).  A formal statement needs an
// improper-integral notion (limits of finite integrals), which the real
// package does not yet provide; see the notes in real/gamma.ac.
