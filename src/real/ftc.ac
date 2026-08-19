from real.real_field import Real
from real.integral import integral, is_integrable, interval_contains, integral_const,
    comparison, constant_integrable, lower_sum, upper_sum, lower_sum_set, upper_sum_set,
    lower_sum_contains, upper_sum_contains, is_partition, partition_start, partition_end,
    partition_mono, partition_point_in_interval, diff_step, telescope, partial_lte,
    interval_inf, interval_sup, interval_inf_spec, interval_sup_spec, interval_set,
    partition_step_lower, partition_step_upper,
    interval_image, interval_contains_left, interval_contains_right, interval_contains_mono,
    sub_nonneg, integral_spec, sup_le_of_upper_bound, set_lower_bound_le_infimum,
    set_infimum_is_lower_bound, set_lower_bound_contains_le, set_supremum_is_upper_bound,
    set_upper_bound_contains_le, image_lower_bound, interval_set_contains_left,
    has_lower_bound, has_upper_bound, is_nonempty, integral_additivity, integral_additivity_symm,
    neg_lte_flip
from real.integral_exp import image_upper_bound
from real.derivative_basic import has_derivative_at, differentiable_at, difference_quotient,
    sub_ne_zero_of_ne, has_derivative_at_unique
from real.derivative_affine_named import affine_real_has_derivative_at
from real.derivative_continuity import derivative_continuous_at, div_mul_cancel_denominator
from real.continuity_affine import affine_real
from real.mean_value import mean_value_theorem_local, continuous_on_closed, is_derivative_on_open,
    secant_slope
from real.real_base import close_imp_bounds, self_close, lt_add_right, gt_zero_imp_pos,
    add_neg_eq_zero, pos_gt_zero, lte_lt_trans, neg_distrib, add_assoc, add_zero_left, abs_neg,
    add_comm, bounds_imp_close, lt_add_converse
from real.real_seq import lt_imp_minus_pos, close_and_lt_imp_close, eps_smaller_than_both
from real.real_field import div_mul_cancel_left, mul_inverse
from real.real_ring import mul_one_right, mul_assoc
from real.derivative_trig import abs_of_nonneg
from real.exp import two_positive, two
from real.am_gm import div_pos_of_pos_pos
from real.convex import neg_div_neg_eq
from real.continuity_base import continuous_at, continuous_condition
from real.supremum import function_image, function_image_contains, is_set_upper_bound,
    is_set_lower_bound, is_set_infimum, is_set_supremum, set_member_le_supremum
from algebra.add_ordered_group import add_le_add_right
from data.basic.set import Set, maps_into_set_image
from data.basic.functions import compose
from list import partial
from nat import Nat, lt_imp_lte_suc
from order import lt_imp_lte, lt_of_lte_of_ne, lte_antisymm, lt_imp_ne_symm, lte_trans,
    closed_interval, lt_trans, lt_of_lte_of_lt, lt_or_lte, lt_of_lt_of_lte, lt_imp_ne
from order_set import closed_interval_set, closed_interval_set_contains_eq
from ordered_field import mul_le_mul_of_nonneg_right, inverse_of_positive_is_positive,
    mul_lt_mul_of_pos_right

numerals Real
numerals Nat

// This file states the Fundamental Theorem of Calculus (FTC) on top of the
// Riemann (Darboux) integral of real/integral.ac:
//   - the boundedness of the integral (integral_bounds),
//   - FTC part 2 (ftc2): if g is differentiable on [a, b] with pointwise
//     derivative f, f is bounded by lb and ub on [a, b], and f is integrable
//     on [a, b], then the integral of f over [a, b] equals g(b) - g(a).  The
//     proof squeezes the Darboux sums of f between the two sides using the
//     mean value theorem on each cell (mean_value_theorem_local).
//   - FTC part 1 (ftc1): if f is continuous at x0 and integrable on every
//     subinterval of [a, x0 + delta0] for some positive delta0, then the
//     integral function F(x) = integral(f, a, x) is differentiable at x0 with
//     derivative f(x0), and hence continuous there (ftc1_continuous).
//   - the constant-integraand cases (ftc1_const, ftc2_const) proved directly.
// The general proofs need interval additivity of the integral
// (integral_additivity in integral.ac) and the mean value theorem
// (mean_value.ac), both now in the library.

/// The integral function F(x) = integral(f, a, x) of f with fixed lower limit a.
define integral_function(f: Real -> Real, a: Real, x: Real) -> Real {
    integral(f, a, x)
}

// ---------------------------------------------------------------------------
// Boundedness of the integral
// ---------------------------------------------------------------------------

/// The constant function m is bounded below by m on every interval.
theorem const_self_lb(m: Real, a: Real, b: Real) {
    forall(t: Real) { interval_contains(a, b, t) implies m <= constant[Real, Real](m, t) }
} by {
    forall(t: Real) {
        if interval_contains(a, b, t) {
            constant[Real, Real](m, t) = m
            m <= constant[Real, Real](m, t)
        }
    }
}

/// The constant function -m is bounded below by -m on every interval.
theorem const_neg_self_lb(m: Real, a: Real, b: Real) {
    forall(t: Real) { interval_contains(a, b, t) implies -m <= constant[Real, Real](-m, t) }
} by {
    forall(t: Real) {
        if interval_contains(a, b, t) {
            constant[Real, Real](-m, t) = -m
            -m <= constant[Real, Real](-m, t)
        }
    }
}

/// The integral of a function bounded below by -m and above by m on [a, b]
/// lies between -m * (b - a) and m * (b - a).  The last two hypotheses are
/// always true (see const_self_lb and const_neg_self_lb); they make the
/// comparison theorem applicable.
theorem integral_bounds(f: Real -> Real, m: Real, a: Real, b: Real) {
    a <= b and is_integrable(f, a, b) and
    (forall(t: Real) { interval_contains(a, b, t) implies -m <= f(t) }) and
    (forall(t: Real) { interval_contains(a, b, t) implies f(t) <= m }) and
    (forall(t: Real) { interval_contains(a, b, t) implies m <= constant[Real, Real](m, t) }) and
    (forall(t: Real) { interval_contains(a, b, t) implies -m <= constant[Real, Real](-m, t) })
    implies
    (-m) * (b - a) <= integral(f, a, b) and integral(f, a, b) <= m * (b - a)
} by {
    if a <= b and is_integrable(f, a, b) and
       (forall(t: Real) { interval_contains(a, b, t) implies -m <= f(t) }) and
       (forall(t: Real) { interval_contains(a, b, t) implies f(t) <= m }) and
       (forall(t: Real) { interval_contains(a, b, t) implies m <= constant[Real, Real](m, t) }) and
       (forall(t: Real) { interval_contains(a, b, t) implies -m <= constant[Real, Real](-m, t) }) {
        // f is bounded above by the constant function m on [a, b].
        forall(t: Real) {
            if interval_contains(a, b, t) {
                forall(t0: Real) {
                    interval_contains(a, b, t0) implies f(t0) <= m
                }
                interval_contains(a, b, t) implies f(t) <= m
                f(t) <= m
                constant[Real, Real](m, t) = m
                f(t) <= constant[Real, Real](m, t)
            }
        }
        constant_integrable(constant[Real, Real](m), m, a, b)
        is_integrable(constant[Real, Real](m), a, b)
        comparison(f, constant[Real, Real](m), a, b, -m, m)
        integral(f, a, b) <= integral(constant[Real, Real](m), a, b)
        integral_const(constant[Real, Real](m), m, a, b)
        integral(constant[Real, Real](m), a, b) = m * (b - a)
        integral(f, a, b) <= m * (b - a)
        // f is bounded below by the constant function -m on [a, b].
        forall(t: Real) {
            if interval_contains(a, b, t) {
                forall(t1: Real) {
                    interval_contains(a, b, t1) implies -m <= f(t1)
                }
                interval_contains(a, b, t) implies -m <= f(t)
                -m <= f(t)
                constant[Real, Real](-m, t) = -m
                constant[Real, Real](-m, t) <= f(t)
            }
        }
        constant_integrable(constant[Real, Real](-m), -m, a, b)
        is_integrable(constant[Real, Real](-m), a, b)
        comparison(constant[Real, Real](-m), f, a, b, -m, -m)
        integral(constant[Real, Real](-m), a, b) <= integral(f, a, b)
        integral_const(constant[Real, Real](-m), -m, a, b)
        integral(constant[Real, Real](-m), a, b) = (-m) * (b - a)
        (-m) * (b - a) <= integral(f, a, b)
        (-m) * (b - a) <= integral(f, a, b) and integral(f, a, b) <= m * (b - a)
    }
}

// ---------------------------------------------------------------------------
// FTC part 2: evaluation of the integral from an antiderivative
// ---------------------------------------------------------------------------

/// A function with a pointwise derivative on [x, y] is continuous on [x, y].
theorem derivative_on_closed_imp_continuous_on_closed(g: Real -> Real, f: Real -> Real, x: Real, y: Real) {
    (forall(t: Real) { interval_contains(x, y, t) implies has_derivative_at(g, t, f(t)) })
    implies continuous_on_closed(g, x, y)
} by {
    if forall(t: Real) { interval_contains(x, y, t) implies has_derivative_at(g, t, f(t)) } {
        forall(t: Real) {
            if closed_interval_set(x, y).contains(t) {
                closed_interval_set_contains_eq(x, y, t)
                closed_interval_set(x, y).contains(t) = closed_interval(x, y, t)
                closed_interval(x, y, t)
                interval_contains(x, y, t)
                forall(t0: Real) {
                    interval_contains(x, y, t0) implies has_derivative_at(g, t0, f(t0))
                }
                interval_contains(x, y, t) implies has_derivative_at(g, t, f(t))
                has_derivative_at(g, t, f(t))
                derivative_continuous_at(g, t, f(t))
                continuous_at(g, t)
            }
        }
        continuous_on_closed(g, x, y) = forall(t: Real) {
            closed_interval_set(x, y).contains(t) implies continuous_at(g, t)
        }
        continuous_on_closed(g, x, y)
    }
}

/// A pointwise derivative on [x, y] restricts to the open interval (x, y).
theorem derivative_on_closed_imp_derivative_on_open(g: Real -> Real, f: Real -> Real, x: Real, y: Real) {
    (forall(t: Real) { interval_contains(x, y, t) implies has_derivative_at(g, t, f(t)) })
    implies is_derivative_on_open(g, f, x, y)
} by {
    if forall(t: Real) { interval_contains(x, y, t) implies has_derivative_at(g, t, f(t)) } {
        forall(t: Real) {
            if x < t and t < y {
                lt_imp_lte(x, t)
                x <= t
                lt_imp_lte(t, y)
                t <= y
                interval_contains(x, y, t)
                forall(t0: Real) {
                    interval_contains(x, y, t0) implies has_derivative_at(g, t0, f(t0))
                }
                interval_contains(x, y, t) implies has_derivative_at(g, t, f(t))
                has_derivative_at(g, t, f(t))
            }
        }
        is_derivative_on_open(g, f, x, y) = forall(t: Real) {
            x < t and t < y implies has_derivative_at(g, t, f(t))
        }
        is_derivative_on_open(g, f, x, y)
    }
}

/// The mean value theorem on a cell of the integral function: if g has
/// pointwise derivative f on [x, y], then the increase of g across the cell is
/// f at some interior point times the width.
theorem ftc2_cell_mvt_result(f: Real -> Real, g: Real -> Real, x: Real, y: Real) {
    x <= y and
    (forall(t: Real) { interval_contains(x, y, t) implies has_derivative_at(g, t, f(t)) })
    implies exists(c: Real) {
        interval_contains(x, y, c) and g(y) - g(x) = f(c) * (y - x)
    }
} by {
    if x <= y and (forall(t: Real) { interval_contains(x, y, t) implies has_derivative_at(g, t, f(t)) }) {
        if x = y {
            interval_contains(x, y, x)
            g(y) - g(x) = g(x) - g(x)
            g(x) - g(x) = Real.0
            y - x = Real.0
            f(x) * (y - x) = f(x) * Real.0
            f(x) * Real.0 = Real.0
            g(y) - g(x) = f(x) * (y - x)
            interval_contains(x, y, x) and g(y) - g(x) = f(x) * (y - x)
            exists(c: Real) {
                interval_contains(x, y, c) and g(y) - g(x) = f(c) * (y - x)
            }
        } else {
            x != y
            lt_of_lte_of_ne[Real](x, y)
            x < y
            derivative_on_closed_imp_continuous_on_closed(g, f, x, y)
            continuous_on_closed(g, x, y)
            derivative_on_closed_imp_derivative_on_open(g, f, x, y)
            is_derivative_on_open(g, f, x, y)
            mean_value_theorem_local(g, f, x, y)
            let c: Real satisfy {
                x < c and c < y and has_derivative_at(g, c, secant_slope(g, x, y))
            }
            x < c and c < y
            has_derivative_at(g, c, secant_slope(g, x, y))
            lt_imp_lte(x, c)
            x <= c
            lt_imp_lte(c, y)
            c <= y
            interval_contains(x, y, c)
            forall(t0: Real) {
                interval_contains(x, y, t0) implies has_derivative_at(g, t0, f(t0))
            }
            interval_contains(x, y, c) implies has_derivative_at(g, c, f(c))
            has_derivative_at(g, c, f(c))
            has_derivative_at_unique(g, c, secant_slope(g, x, y), f(c))
            secant_slope(g, x, y) = f(c)
            secant_slope(g, x, y) = (g(y) - g(x)) / (y - x)
            (g(y) - g(x)) / (y - x) = f(c)
            lt_imp_ne_symm(x, y)
            y != x
            sub_ne_zero_of_ne(y, x)
            y - x != Real.0
            div_mul_cancel_denominator(g(y) - g(x), y - x)
            ((g(y) - g(x)) / (y - x)) * (y - x) = g(y) - g(x)
            ((g(y) - g(x)) / (y - x)) * (y - x) = f(c) * (y - x)
            g(y) - g(x) = f(c) * (y - x)
            interval_contains(x, y, c) and g(y) - g(x) = f(c) * (y - x)
            exists(c2: Real) {
                interval_contains(x, y, c2) and g(y) - g(x) = f(c2) * (y - x)
            }
        }
    }
}

/// The lower Darboux step of the integral of f on a cell is at most the
/// increase of any antiderivative g of f across the cell.
theorem ftc2_cell_lower(f: Real -> Real, g: Real -> Real, x: Real, y: Real, lb: Real, ub: Real) {
    x <= y and
    (forall(t: Real) { interval_contains(x, y, t) implies has_derivative_at(g, t, f(t)) }) and
    (forall(t: Real) { interval_contains(x, y, t) implies lb <= f(t) }) and
    (forall(t: Real) { interval_contains(x, y, t) implies f(t) <= ub })
    implies
    interval_inf(f, x, y) * (y - x) <= g(y) - g(x)
} by {
    if x <= y and
       (forall(t: Real) { interval_contains(x, y, t) implies has_derivative_at(g, t, f(t)) }) and
       (forall(t: Real) { interval_contains(x, y, t) implies lb <= f(t) }) and
       (forall(t: Real) { interval_contains(x, y, t) implies f(t) <= ub }) {
        ftc2_cell_mvt_result(f, g, x, y)
        let c: Real satisfy {
            interval_contains(x, y, c) and g(y) - g(x) = f(c) * (y - x)
        }
        interval_contains(x, y, c)
        g(y) - g(x) = f(c) * (y - x)
        // interval_inf(f, x, y) <= f(c)
        interval_set_contains_left(x, y)
        is_nonempty(interval_set(x, y))
        image_lower_bound(f, x, y, lb)
        is_set_lower_bound(interval_image(f, x, y), lb)
        exists(b: Real) {
            is_set_lower_bound(interval_image(f, x, y), b)
        }
        has_lower_bound(interval_image(f, x, y))
        is_nonempty(interval_set(x, y)) and has_lower_bound(interval_image(f, x, y))
        interval_inf_spec(f, x, y)
        is_set_infimum(interval_image(f, x, y), interval_inf(f, x, y))
        set_infimum_is_lower_bound(interval_image(f, x, y), interval_inf(f, x, y))
        is_set_lower_bound(interval_image(f, x, y), interval_inf(f, x, y))
        interval_set(x, y).contains(c) = interval_contains(x, y, c)
        interval_contains(x, y, c)
        interval_set(x, y).contains(c)
        maps_into_set_image(interval_set(x, y), f, c)
        function_image(f, interval_set(x, y)).contains(f(c))
        interval_image(f, x, y).contains(f(c))
        set_lower_bound_contains_le(interval_image(f, x, y), interval_inf(f, x, y), f(c))
        interval_inf(f, x, y) <= f(c)
        sub_nonneg(x, y)
        Real.0 <= y - x
        mul_le_mul_of_nonneg_right(interval_inf(f, x, y), f(c), y - x)
        interval_inf(f, x, y) * (y - x) <= f(c) * (y - x)
        f(c) * (y - x) = g(y) - g(x)
        interval_inf(f, x, y) * (y - x) <= g(y) - g(x)
    }
}

/// The upper Darboux step of the integral of f on a cell is at least the
/// increase of any antiderivative g of f across the cell.
theorem ftc2_cell_upper(f: Real -> Real, g: Real -> Real, x: Real, y: Real, lb: Real, ub: Real) {
    x <= y and
    (forall(t: Real) { interval_contains(x, y, t) implies has_derivative_at(g, t, f(t)) }) and
    (forall(t: Real) { interval_contains(x, y, t) implies lb <= f(t) }) and
    (forall(t: Real) { interval_contains(x, y, t) implies f(t) <= ub })
    implies
    g(y) - g(x) <= interval_sup(f, x, y) * (y - x)
} by {
    if x <= y and
       (forall(t: Real) { interval_contains(x, y, t) implies has_derivative_at(g, t, f(t)) }) and
       (forall(t: Real) { interval_contains(x, y, t) implies lb <= f(t) }) and
       (forall(t: Real) { interval_contains(x, y, t) implies f(t) <= ub }) {
        ftc2_cell_mvt_result(f, g, x, y)
        let c: Real satisfy {
            interval_contains(x, y, c) and g(y) - g(x) = f(c) * (y - x)
        }
        interval_contains(x, y, c)
        g(y) - g(x) = f(c) * (y - x)
        // f(c) <= interval_sup(f, x, y)
        interval_set_contains_left(x, y)
        is_nonempty(interval_set(x, y))
        image_upper_bound(f, x, y, ub)
        is_set_upper_bound(interval_image(f, x, y), ub)
        exists(b: Real) {
            is_set_upper_bound(interval_image(f, x, y), b)
        }
        has_upper_bound(interval_image(f, x, y))
        is_nonempty(interval_set(x, y)) and has_upper_bound(interval_image(f, x, y))
        interval_sup_spec(f, x, y)
        is_set_supremum(interval_image(f, x, y), interval_sup(f, x, y))
        set_supremum_is_upper_bound(interval_image(f, x, y), interval_sup(f, x, y))
        is_set_upper_bound(interval_image(f, x, y), interval_sup(f, x, y))
        interval_set(x, y).contains(c) = interval_contains(x, y, c)
        interval_contains(x, y, c)
        interval_set(x, y).contains(c)
        maps_into_set_image(interval_set(x, y), f, c)
        function_image(f, interval_set(x, y)).contains(f(c))
        interval_image(f, x, y).contains(f(c))
        set_upper_bound_contains_le(interval_image(f, x, y), interval_sup(f, x, y), f(c))
        f(c) <= interval_sup(f, x, y)
        sub_nonneg(x, y)
        Real.0 <= y - x
        mul_le_mul_of_nonneg_right(f(c), interval_sup(f, x, y), y - x)
        f(c) * (y - x) <= interval_sup(f, x, y) * (y - x)
        g(y) - g(x) = f(c) * (y - x)
        g(y) - g(x) <= interval_sup(f, x, y) * (y - x)
    }
}

/// A pointwise derivative on [a, b] restricts to every partition cell of a
/// partition of [a, b].
theorem partition_cell_has_derivative(f: Real -> Real, g: Real -> Real, p: Nat -> Real, a: Real, b: Real, n: Nat, k: Nat) {
    is_partition(p, a, b, n) and k < n and
    (forall(t: Real) { interval_contains(a, b, t) implies has_derivative_at(g, t, f(t)) })
    implies forall(t: Real) { interval_contains(p(k), p(k + 1), t) implies has_derivative_at(g, t, f(t)) }
} by {
    if is_partition(p, a, b, n) and k < n and
       (forall(t: Real) { interval_contains(a, b, t) implies has_derivative_at(g, t, f(t)) }) {
        lt_imp_lte_suc(k, n)
        k + 1 <= n
        k <= k + 1
        lte_trans[Nat](k, k + 1, n)
        partition_mono(p, a, b, n, k, k + 1)
        p(k) <= p(k + 1)
        partition_point_in_interval(p, a, b, n, k)
        interval_contains(a, b, p(k))
        partition_point_in_interval(p, a, b, n, k + 1)
        interval_contains(a, b, p(k + 1))
        forall(t: Real) {
            if interval_contains(p(k), p(k + 1), t) {
                interval_contains_left(p(k), p(k + 1), t)
                p(k) <= t
                interval_contains_right(p(k), p(k + 1), t)
                t <= p(k + 1)
                interval_contains_mono(a, b, p(k), p(k + 1), t)
                interval_contains(a, b, t)
                forall(t0: Real) {
                    interval_contains(a, b, t0) implies has_derivative_at(g, t0, f(t0))
                }
                interval_contains(a, b, t) implies has_derivative_at(g, t, f(t))
                has_derivative_at(g, t, f(t))
            }
        }
        forall(t: Real) {
            interval_contains(p(k), p(k + 1), t) implies has_derivative_at(g, t, f(t))
        }
    }
}

/// A pointwise lower bound on [a, b] restricts to every partition cell of a
/// partition of [a, b].
theorem partition_cell_lb(f: Real -> Real, lb: Real, p: Nat -> Real, a: Real, b: Real, n: Nat, k: Nat) {
    is_partition(p, a, b, n) and k < n and
    (forall(t: Real) { interval_contains(a, b, t) implies lb <= f(t) })
    implies forall(t: Real) { interval_contains(p(k), p(k + 1), t) implies lb <= f(t) }
} by {
    if is_partition(p, a, b, n) and k < n and
       (forall(t: Real) { interval_contains(a, b, t) implies lb <= f(t) }) {
        lt_imp_lte_suc(k, n)
        k + 1 <= n
        k <= k + 1
        lte_trans[Nat](k, k + 1, n)
        partition_mono(p, a, b, n, k, k + 1)
        p(k) <= p(k + 1)
        partition_point_in_interval(p, a, b, n, k)
        interval_contains(a, b, p(k))
        partition_point_in_interval(p, a, b, n, k + 1)
        interval_contains(a, b, p(k + 1))
        forall(t: Real) {
            if interval_contains(p(k), p(k + 1), t) {
                interval_contains_left(p(k), p(k + 1), t)
                p(k) <= t
                interval_contains_right(p(k), p(k + 1), t)
                t <= p(k + 1)
                interval_contains_mono(a, b, p(k), p(k + 1), t)
                interval_contains(a, b, t)
                forall(t1: Real) {
                    interval_contains(a, b, t1) implies lb <= f(t1)
                }
                interval_contains(a, b, t) implies lb <= f(t)
                lb <= f(t)
            }
        }
        forall(t: Real) {
            interval_contains(p(k), p(k + 1), t) implies lb <= f(t)
        }
    }
}

/// A pointwise upper bound on [a, b] restricts to every partition cell of a
/// partition of [a, b].
theorem partition_cell_ub(f: Real -> Real, ub: Real, p: Nat -> Real, a: Real, b: Real, n: Nat, k: Nat) {
    is_partition(p, a, b, n) and k < n and
    (forall(t: Real) { interval_contains(a, b, t) implies f(t) <= ub })
    implies forall(t: Real) { interval_contains(p(k), p(k + 1), t) implies f(t) <= ub }
} by {
    if is_partition(p, a, b, n) and k < n and
       (forall(t: Real) { interval_contains(a, b, t) implies f(t) <= ub }) {
        lt_imp_lte_suc(k, n)
        k + 1 <= n
        k <= k + 1
        lte_trans[Nat](k, k + 1, n)
        partition_mono(p, a, b, n, k, k + 1)
        p(k) <= p(k + 1)
        partition_point_in_interval(p, a, b, n, k)
        interval_contains(a, b, p(k))
        partition_point_in_interval(p, a, b, n, k + 1)
        interval_contains(a, b, p(k + 1))
        forall(t: Real) {
            if interval_contains(p(k), p(k + 1), t) {
                interval_contains_left(p(k), p(k + 1), t)
                p(k) <= t
                interval_contains_right(p(k), p(k + 1), t)
                t <= p(k + 1)
                interval_contains_mono(a, b, p(k), p(k + 1), t)
                interval_contains(a, b, t)
                forall(t2: Real) {
                    interval_contains(a, b, t2) implies f(t2) <= ub
                }
                interval_contains(a, b, t) implies f(t) <= ub
                f(t) <= ub
            }
        }
        forall(t: Real) {
            interval_contains(p(k), p(k + 1), t) implies f(t) <= ub
        }
    }
}

/// On every cell of a partition of [a, b], the lower Darboux step of the
/// integral of f is at most the increase of any antiderivative g of f.
theorem ftc2_partition_cell_lower(f: Real -> Real, g: Real -> Real, p: Nat -> Real, a: Real, b: Real, n: Nat, k: Nat, lb: Real, ub: Real) {
    is_partition(p, a, b, n) and k < n and
    (forall(t: Real) { interval_contains(a, b, t) implies has_derivative_at(g, t, f(t)) }) and
    (forall(t: Real) { interval_contains(a, b, t) implies lb <= f(t) }) and
    (forall(t: Real) { interval_contains(a, b, t) implies f(t) <= ub })
    implies
    interval_inf(f, p(k), p(k + 1)) * (p(k + 1) - p(k)) <= g(p(k + 1)) - g(p(k))
} by {
    if is_partition(p, a, b, n) and k < n and
       (forall(t: Real) { interval_contains(a, b, t) implies has_derivative_at(g, t, f(t)) }) and
       (forall(t: Real) { interval_contains(a, b, t) implies lb <= f(t) }) and
       (forall(t: Real) { interval_contains(a, b, t) implies f(t) <= ub }) {
        lt_imp_lte_suc(k, n)
        k + 1 <= n
        k <= k + 1
        lte_trans[Nat](k, k + 1, n)
        partition_mono(p, a, b, n, k, k + 1)
        p(k) <= p(k + 1)
        partition_cell_has_derivative(f, g, p, a, b, n, k)
        forall(t: Real) {
            interval_contains(p(k), p(k + 1), t) implies has_derivative_at(g, t, f(t))
        }
        partition_cell_lb(f, lb, p, a, b, n, k)
        forall(t: Real) {
            interval_contains(p(k), p(k + 1), t) implies lb <= f(t)
        }
        ftc2_cell_mvt_result(f, g, p(k), p(k + 1))
        let c: Real satisfy {
            interval_contains(p(k), p(k + 1), c) and
                g(p(k + 1)) - g(p(k)) = f(c) * (p(k + 1) - p(k))
        }
        interval_contains(p(k), p(k + 1), c)
        g(p(k + 1)) - g(p(k)) = f(c) * (p(k + 1) - p(k))
        // interval_inf(f, p(k), p(k + 1)) <= f(c)
        interval_set_contains_left(p(k), p(k + 1))
        is_nonempty(interval_set(p(k), p(k + 1)))
        image_lower_bound(f, p(k), p(k + 1), lb)
        is_set_lower_bound(interval_image(f, p(k), p(k + 1)), lb)
        exists(b0: Real) {
            is_set_lower_bound(interval_image(f, p(k), p(k + 1)), b0)
        }
        has_lower_bound(interval_image(f, p(k), p(k + 1)))
        is_nonempty(interval_set(p(k), p(k + 1))) and has_lower_bound(interval_image(f, p(k), p(k + 1)))
        interval_inf_spec(f, p(k), p(k + 1))
        is_set_infimum(interval_image(f, p(k), p(k + 1)), interval_inf(f, p(k), p(k + 1)))
        set_infimum_is_lower_bound(interval_image(f, p(k), p(k + 1)), interval_inf(f, p(k), p(k + 1)))
        is_set_lower_bound(interval_image(f, p(k), p(k + 1)), interval_inf(f, p(k), p(k + 1)))
        interval_set(p(k), p(k + 1)).contains(c) = interval_contains(p(k), p(k + 1), c)
        interval_contains(p(k), p(k + 1), c)
        interval_set(p(k), p(k + 1)).contains(c)
        maps_into_set_image(interval_set(p(k), p(k + 1)), f, c)
        function_image(f, interval_set(p(k), p(k + 1))).contains(f(c))
        interval_image(f, p(k), p(k + 1)).contains(f(c))
        set_lower_bound_contains_le(interval_image(f, p(k), p(k + 1)), interval_inf(f, p(k), p(k + 1)), f(c))
        interval_inf(f, p(k), p(k + 1)) <= f(c)
        sub_nonneg(p(k), p(k + 1))
        Real.0 <= p(k + 1) - p(k)
        mul_le_mul_of_nonneg_right(interval_inf(f, p(k), p(k + 1)), f(c), p(k + 1) - p(k))
        interval_inf(f, p(k), p(k + 1)) * (p(k + 1) - p(k)) <= f(c) * (p(k + 1) - p(k))
        f(c) * (p(k + 1) - p(k)) = g(p(k + 1)) - g(p(k))
        interval_inf(f, p(k), p(k + 1)) * (p(k + 1) - p(k)) <= g(p(k + 1)) - g(p(k))
    }
}

/// On every cell of a partition of [a, b], the increase of any antiderivative
/// g of f is at most the upper Darboux step of the integral of f.
theorem ftc2_partition_cell_upper(f: Real -> Real, g: Real -> Real, p: Nat -> Real, a: Real, b: Real, n: Nat, k: Nat, lb: Real, ub: Real) {
    is_partition(p, a, b, n) and k < n and
    (forall(t: Real) { interval_contains(a, b, t) implies has_derivative_at(g, t, f(t)) }) and
    (forall(t: Real) { interval_contains(a, b, t) implies lb <= f(t) }) and
    (forall(t: Real) { interval_contains(a, b, t) implies f(t) <= ub })
    implies
    g(p(k + 1)) - g(p(k)) <= interval_sup(f, p(k), p(k + 1)) * (p(k + 1) - p(k))
} by {
    if is_partition(p, a, b, n) and k < n and
       (forall(t: Real) { interval_contains(a, b, t) implies has_derivative_at(g, t, f(t)) }) and
       (forall(t: Real) { interval_contains(a, b, t) implies lb <= f(t) }) and
       (forall(t: Real) { interval_contains(a, b, t) implies f(t) <= ub }) {
        lt_imp_lte_suc(k, n)
        k + 1 <= n
        k <= k + 1
        lte_trans[Nat](k, k + 1, n)
        partition_mono(p, a, b, n, k, k + 1)
        p(k) <= p(k + 1)
        partition_cell_has_derivative(f, g, p, a, b, n, k)
        forall(t: Real) {
            interval_contains(p(k), p(k + 1), t) implies has_derivative_at(g, t, f(t))
        }
        partition_cell_ub(f, ub, p, a, b, n, k)
        forall(t: Real) {
            interval_contains(p(k), p(k + 1), t) implies f(t) <= ub
        }
        ftc2_cell_mvt_result(f, g, p(k), p(k + 1))
        let c: Real satisfy {
            interval_contains(p(k), p(k + 1), c) and
                g(p(k + 1)) - g(p(k)) = f(c) * (p(k + 1) - p(k))
        }
        interval_contains(p(k), p(k + 1), c)
        g(p(k + 1)) - g(p(k)) = f(c) * (p(k + 1) - p(k))
        // f(c) <= interval_sup(f, p(k), p(k + 1))
        interval_set_contains_left(p(k), p(k + 1))
        is_nonempty(interval_set(p(k), p(k + 1)))
        image_upper_bound(f, p(k), p(k + 1), ub)
        is_set_upper_bound(interval_image(f, p(k), p(k + 1)), ub)
        exists(b0: Real) {
            is_set_upper_bound(interval_image(f, p(k), p(k + 1)), b0)
        }
        has_upper_bound(interval_image(f, p(k), p(k + 1)))
        is_nonempty(interval_set(p(k), p(k + 1))) and has_upper_bound(interval_image(f, p(k), p(k + 1)))
        interval_sup_spec(f, p(k), p(k + 1))
        is_set_supremum(interval_image(f, p(k), p(k + 1)), interval_sup(f, p(k), p(k + 1)))
        set_supremum_is_upper_bound(interval_image(f, p(k), p(k + 1)), interval_sup(f, p(k), p(k + 1)))
        is_set_upper_bound(interval_image(f, p(k), p(k + 1)), interval_sup(f, p(k), p(k + 1)))
        interval_set(p(k), p(k + 1)).contains(c) = interval_contains(p(k), p(k + 1), c)
        interval_contains(p(k), p(k + 1), c)
        interval_set(p(k), p(k + 1)).contains(c)
        maps_into_set_image(interval_set(p(k), p(k + 1)), f, c)
        function_image(f, interval_set(p(k), p(k + 1))).contains(f(c))
        interval_image(f, p(k), p(k + 1)).contains(f(c))
        set_upper_bound_contains_le(interval_image(f, p(k), p(k + 1)), interval_sup(f, p(k), p(k + 1)), f(c))
        f(c) <= interval_sup(f, p(k), p(k + 1))
        sub_nonneg(p(k), p(k + 1))
        Real.0 <= p(k + 1) - p(k)
        mul_le_mul_of_nonneg_right(f(c), interval_sup(f, p(k), p(k + 1)), p(k + 1) - p(k))
        f(c) * (p(k + 1) - p(k)) <= interval_sup(f, p(k), p(k + 1)) * (p(k + 1) - p(k))
        g(p(k + 1)) - g(p(k)) = f(c) * (p(k + 1) - p(k))
        g(p(k + 1)) - g(p(k)) <= interval_sup(f, p(k), p(k + 1)) * (p(k + 1) - p(k))
    }
}

/// The lower Darboux sum of f over any partition of [a, b] is at most the
/// increase of any antiderivative g of f across [a, b].
theorem ftc2_lower_sum_le(f: Real -> Real, g: Real -> Real, p: Nat -> Real, a: Real, b: Real, n: Nat, lb: Real, ub: Real) {
    a <= b and is_partition(p, a, b, n) and
    (forall(t: Real) { interval_contains(a, b, t) implies has_derivative_at(g, t, f(t)) }) and
    (forall(t: Real) { interval_contains(a, b, t) implies lb <= f(t) }) and
    (forall(t: Real) { interval_contains(a, b, t) implies f(t) <= ub })
    implies
    lower_sum(f, p, n) <= g(b) - g(a)
} by {
    if a <= b and is_partition(p, a, b, n) and
       (forall(t: Real) { interval_contains(a, b, t) implies has_derivative_at(g, t, f(t)) }) and
       (forall(t: Real) { interval_contains(a, b, t) implies lb <= f(t) }) and
       (forall(t: Real) { interval_contains(a, b, t) implies f(t) <= ub }) {
        forall(k: Nat) {
            if k < n {
                lt_imp_lte_suc(k, n)
                k + 1 <= n
                k <= k + 1
                lte_trans[Nat](k, k + 1, n)
                partition_mono(p, a, b, n, k, k + 1)
                p(k) <= p(k + 1)
                ftc2_partition_cell_lower(f, g, p, a, b, n, k, lb, ub)
                interval_inf(f, p(k), p(k + 1)) * (p(k + 1) - p(k)) <= g(p(k + 1)) - g(p(k))
                partition_step_lower(f, p, k) = interval_inf(f, p(k), p(k + 1)) * diff_step(p, k)
                diff_step(p, k) = p(k + 1) - p(k)
                interval_inf(f, p(k), p(k + 1)) * diff_step(p, k) = interval_inf(f, p(k), p(k + 1)) * (p(k + 1) - p(k))
                partition_step_lower(f, p, k) = interval_inf(f, p(k), p(k + 1)) * (p(k + 1) - p(k))
                partition_step_lower(f, p, k) <= g(p(k + 1)) - g(p(k))
                diff_step(compose(g, p), k) = g(p(k + 1)) - g(p(k))
                partition_step_lower(f, p, k) <= diff_step(compose(g, p), k)
            }
        }
        partial_lte(partition_step_lower(f, p), diff_step(compose(g, p)), n)
        partial(partition_step_lower(f, p), n) <= partial(diff_step(compose(g, p)), n)
        telescope(compose(g, p), n)
        partial(diff_step(compose(g, p)), n) = compose(g, p)(n) - compose(g, p)(Nat.0)
        compose(g, p, n) = g(p(n))
        compose(g, p, Nat.0) = g(p(Nat.0))
        partial(diff_step(compose(g, p)), n) = g(p(n)) - g(p(Nat.0))
        partition_start(p, a, b, n)
        p(Nat.0) = a
        partition_end(p, a, b, n)
        p(n) = b
        g(p(n)) - g(p(Nat.0)) = g(b) - g(a)
        partial(diff_step(compose(g, p)), n) = g(b) - g(a)
        partial(partition_step_lower(f, p), n) <= g(b) - g(a)
        lower_sum(f, p, n) = partial(partition_step_lower(f, p), n)
        lower_sum(f, p, n) <= g(b) - g(a)
    }
}

/// The upper Darboux sum of f over any partition of [a, b] is at least the
/// increase of any antiderivative g of f across [a, b].
theorem ftc2_upper_sum_ge(f: Real -> Real, g: Real -> Real, p: Nat -> Real, a: Real, b: Real, n: Nat, lb: Real, ub: Real) {
    a <= b and is_partition(p, a, b, n) and
    (forall(t: Real) { interval_contains(a, b, t) implies has_derivative_at(g, t, f(t)) }) and
    (forall(t: Real) { interval_contains(a, b, t) implies lb <= f(t) }) and
    (forall(t: Real) { interval_contains(a, b, t) implies f(t) <= ub })
    implies
    g(b) - g(a) <= upper_sum(f, p, n)
} by {
    if a <= b and is_partition(p, a, b, n) and
       (forall(t: Real) { interval_contains(a, b, t) implies has_derivative_at(g, t, f(t)) }) and
       (forall(t: Real) { interval_contains(a, b, t) implies lb <= f(t) }) and
       (forall(t: Real) { interval_contains(a, b, t) implies f(t) <= ub }) {
        forall(k: Nat) {
            if k < n {
                lt_imp_lte_suc(k, n)
                k + 1 <= n
                k <= k + 1
                lte_trans[Nat](k, k + 1, n)
                partition_mono(p, a, b, n, k, k + 1)
                p(k) <= p(k + 1)
                ftc2_partition_cell_upper(f, g, p, a, b, n, k, lb, ub)
                g(p(k + 1)) - g(p(k)) <= interval_sup(f, p(k), p(k + 1)) * (p(k + 1) - p(k))
                partition_step_upper(f, p, k) = interval_sup(f, p(k), p(k + 1)) * diff_step(p, k)
                diff_step(p, k) = p(k + 1) - p(k)
                interval_sup(f, p(k), p(k + 1)) * diff_step(p, k) = interval_sup(f, p(k), p(k + 1)) * (p(k + 1) - p(k))
                partition_step_upper(f, p, k) = interval_sup(f, p(k), p(k + 1)) * (p(k + 1) - p(k))
                g(p(k + 1)) - g(p(k)) <= partition_step_upper(f, p, k)
                diff_step(compose(g, p), k) = g(p(k + 1)) - g(p(k))
                diff_step(compose(g, p), k) <= partition_step_upper(f, p, k)
            }
        }
        partial_lte(diff_step(compose(g, p)), partition_step_upper(f, p), n)
        partial(diff_step(compose(g, p)), n) <= partial(partition_step_upper(f, p), n)
        telescope(compose(g, p), n)
        partial(diff_step(compose(g, p)), n) = compose(g, p)(n) - compose(g, p)(Nat.0)
        compose(g, p, n) = g(p(n))
        compose(g, p, Nat.0) = g(p(Nat.0))
        partial(diff_step(compose(g, p)), n) = g(p(n)) - g(p(Nat.0))
        partition_start(p, a, b, n)
        p(Nat.0) = a
        partition_end(p, a, b, n)
        p(n) = b
        g(p(n)) - g(p(Nat.0)) = g(b) - g(a)
        partial(diff_step(compose(g, p)), n) = g(b) - g(a)
        g(b) - g(a) <= partial(partition_step_upper(f, p), n)
        upper_sum(f, p, n) = partial(partition_step_upper(f, p), n)
        g(b) - g(a) <= upper_sum(f, p, n)
    }
}

/// FTC part 2 (general form): if g is differentiable on [a, b] with pointwise
/// derivative f, f is bounded on [a, b] by lb and ub, and f is integrable on
/// [a, b], then the integral of f over [a, b] equals g(b) - g(a).
theorem ftc2(f: Real -> Real, g: Real -> Real, a: Real, b: Real, lb: Real, ub: Real) {
    a <= b and is_integrable(f, a, b) and
    (forall(t: Real) { interval_contains(a, b, t) implies has_derivative_at(g, t, f(t)) }) and
    (forall(t: Real) { interval_contains(a, b, t) implies lb <= f(t) }) and
    (forall(t: Real) { interval_contains(a, b, t) implies f(t) <= ub })
    implies
    integral(f, a, b) = g(b) - g(a)
} by {
    if a <= b and is_integrable(f, a, b) and
       (forall(t: Real) { interval_contains(a, b, t) implies has_derivative_at(g, t, f(t)) }) and
       (forall(t: Real) { interval_contains(a, b, t) implies lb <= f(t) }) and
       (forall(t: Real) { interval_contains(a, b, t) implies f(t) <= ub }) {
        integral_spec(f, a, b)
        is_set_supremum(lower_sum_set(f, a, b), integral(f, a, b)) and
            is_set_infimum(upper_sum_set(f, a, b), integral(f, a, b))
        is_set_supremum(lower_sum_set(f, a, b), integral(f, a, b))
        is_set_infimum(upper_sum_set(f, a, b), integral(f, a, b))
        // g(b) - g(a) is an upper bound of the lower sums of f.
        forall(x: Real) {
            if lower_sum_set(f, a, b).contains(x) {
                lower_sum_set(f, a, b).contains(x) = lower_sum_contains(f, a, b, x)
                lower_sum_contains(f, a, b, x)
                let (p: Nat -> Real, n: Nat) satisfy {
                    is_partition(p, a, b, n) and x = lower_sum(f, p, n)
                }
                is_partition(p, a, b, n)
                ftc2_lower_sum_le(f, g, p, a, b, n, lb, ub)
                lower_sum(f, p, n) <= g(b) - g(a)
                x = lower_sum(f, p, n)
                x <= g(b) - g(a)
            }
        }
        is_set_upper_bound(lower_sum_set(f, a, b), g(b) - g(a))
        sup_le_of_upper_bound(lower_sum_set(f, a, b), integral(f, a, b), g(b) - g(a))
        integral(f, a, b) <= g(b) - g(a)
        // g(b) - g(a) is a lower bound of the upper sums of f.
        forall(x: Real) {
            if upper_sum_set(f, a, b).contains(x) {
                upper_sum_set(f, a, b).contains(x) = upper_sum_contains(f, a, b, x)
                upper_sum_contains(f, a, b, x)
                let (p: Nat -> Real, n: Nat) satisfy {
                    is_partition(p, a, b, n) and x = upper_sum(f, p, n)
                }
                is_partition(p, a, b, n)
                ftc2_upper_sum_ge(f, g, p, a, b, n, lb, ub)
                g(b) - g(a) <= upper_sum(f, p, n)
                x = upper_sum(f, p, n)
                g(b) - g(a) <= x
            }
        }
        is_set_lower_bound(upper_sum_set(f, a, b), g(b) - g(a))
        set_lower_bound_le_infimum(upper_sum_set(f, a, b), integral(f, a, b), g(b) - g(a))
        g(b) - g(a) <= integral(f, a, b)
        lte_antisymm[Real](integral(f, a, b), g(b) - g(a))
        integral(f, a, b) = g(b) - g(a)
    }
}

// ---------------------------------------------------------------------------
// FTC part 2 for constant integrands
// ---------------------------------------------------------------------------

/// FTC part 2 for constant integrands: if f is constantly c on [a, b], then
/// the integral of f over [a, b] is g(b) - g(a) for every affine
/// antiderivative g(x) = c * x + d.
theorem ftc2_const(f: Real -> Real, c: Real, a: Real, b: Real, d: Real) {
    a <= b and (forall(t: Real) { interval_contains(a, b, t) implies f(t) = c })
    implies
    integral(f, a, b) = affine_real(c, d, b) - affine_real(c, d, a)
} by {
    if a <= b and (forall(t: Real) { interval_contains(a, b, t) implies f(t) = c }) {
        integral_const(f, c, a, b)
        integral(f, a, b) = c * (b - a)
        c * (b - a) = c * b - c * a
        affine_real(c, d, b) = c * b + d
        affine_real(c, d, a) = c * a + d
        (c * b + d) - (c * a + d) = c * b + d - c * a - d
        c * b + d - c * a - d = c * b - c * a + d - d
        c * b - c * a + d - d = c * b - c * a
        affine_real(c, d, b) - affine_real(c, d, a) = c * b - c * a
        integral(f, a, b) = affine_real(c, d, b) - affine_real(c, d, a)
    }
}

/// The affine antiderivative of the constant function c has derivative c at
/// every point.
theorem ftc2_const_derivative(c: Real, d: Real, x0: Real) {
    has_derivative_at(affine_real(c, d), x0, c)
} by {
    affine_real_has_derivative_at(c, d, x0)
    has_derivative_at(affine_real(c, d), x0, c)
}

// ---------------------------------------------------------------------------
// FTC part 1 for constant integrands
// ---------------------------------------------------------------------------

/// The integral function of a function f constantly equal to c agrees with the
/// affine function x -> c * (x - a) on the half-line [a, infinity).
theorem integral_function_eq_affine(f: Real -> Real, c: Real, a: Real, x: Real) {
    (forall(t: Real) { f(t) = c }) and a <= x
    implies
    integral_function(f, a, x) = affine_real(c, -(c * a), x)
} by {
    if (forall(t: Real) { f(t) = c }) and a <= x {
        forall(t: Real) {
            if interval_contains(a, x, t) {
                forall(t0: Real) { f(t0) = c }
                f(t) = c
            }
        }
        integral_const(f, c, a, x)
        integral(f, a, x) = c * (x - a)
        affine_real(c, -(c * a), x) = c * x + -(c * a)
        c * x + -(c * a) = c * (x - a)
        affine_real(c, -(c * a), x) = c * (x - a)
        integral(f, a, x) = affine_real(c, -(c * a), x)
        integral_function(f, a, x) = integral(f, a, x)
        integral_function(f, a, x) = affine_real(c, -(c * a), x)
    }
}

/// The difference quotient of an affine function at any point distinct from the
/// base point equals its slope.
theorem difference_quotient_affine_eq_slope(c: Real, d: Real, x0: Real, x: Real) {
    x != x0 implies difference_quotient(affine_real(c, d), x0, x) = c
} by {
    if x != x0 {
        sub_ne_zero_of_ne(x, x0)
        x - x0 != Real.0
        affine_real(c, d, x) = c * x + d
        affine_real(c, d, x0) = c * x0 + d
        difference_quotient(affine_real(c, d), x0, x) = ((c * x + d) - (c * x0 + d)) / (x - x0)
        (c * x + d) - (c * x0 + d) = c * x + d - c * x0 - d
        c * x + d - c * x0 - d = c * x - c * x0 + d - d
        c * x - c * x0 + d - d = c * x - c * x0
        c * x - c * x0 = c * (x - x0)
        difference_quotient(affine_real(c, d), x0, x) = (c * (x - x0)) / (x - x0)
        c * (x - x0) = (x - x0) * c
        difference_quotient(affine_real(c, d), x0, x) = ((x - x0) * c) / (x - x0)
        div_mul_cancel_left(x - x0, c)
        ((x - x0) * c) / (x - x0) = c
        difference_quotient(affine_real(c, d), x0, x) = c
    }
}

/// In a punctured neighborhood of a point x0 above a, the difference quotient
/// of the integral function of a function constantly equal to c equals c.
theorem integral_function_difference_quotient_const(
    f: Real -> Real, c: Real, a: Real, x0: Real, x: Real
) {
    (forall(t: Real) { f(t) = c }) and a < x0 and x != x0 and x.is_close(x0, x0 - a)
    implies
    difference_quotient(integral_function(f, a), x0, x) = c
} by {
    if (forall(t: Real) { f(t) = c }) and a < x0 and x != x0 and x.is_close(x0, x0 - a) {
        close_imp_bounds(x, x0, x0 - a)
        x0 - (x0 - a) < x and x < x0 + (x0 - a) and x > x0 - (x0 - a) and x0 < x + (x0 - a)
        x0 - (x0 - a) < x
        -(x0 - a) = -x0 + a
        x0 - (x0 - a) = x0 + (-x0 + a)
        x0 + (-x0 + a) = x0 - x0 + a
        x0 - x0 + a = a
        x0 - (x0 - a) = a
        a < x
        lt_imp_lte(a, x)
        a <= x
        integral_function_eq_affine(f, c, a, x)
        integral_function(f, a, x) = affine_real(c, -(c * a), x)
        lt_imp_lte(a, x0)
        a <= x0
        integral_function_eq_affine(f, c, a, x0)
        integral_function(f, a, x0) = affine_real(c, -(c * a), x0)
        difference_quotient_affine_eq_slope(c, -(c * a), x0, x)
        difference_quotient(affine_real(c, -(c * a)), x0, x) = c
        difference_quotient(integral_function(f, a), x0, x) = c
    }
}

/// FTC part 1 for constant integrands: the integral function of a function f
/// constantly equal to c is differentiable at every point x0 above a, with
/// derivative c.
theorem ftc1_const(f: Real -> Real, c: Real, a: Real, x0: Real) {
    (forall(t: Real) { f(t) = c }) and a < x0
    implies
    has_derivative_at(integral_function(f, a), x0, c)
} by {
    if (forall(t: Real) { f(t) = c }) and a < x0 {
        lt_add_right(a, x0, -a)
        a + -a < x0 + -a
        add_neg_eq_zero(a)
        a + -a = Real.0
        Real.0 < x0 + -a
        x0 + -a = x0 - a
        Real.0 < x0 - a
        gt_zero_imp_pos(x0 - a)
        (x0 - a).is_positive
        forall(eps: Real) {
            if eps.is_positive {
                forall(x: Real) {
                    if x != x0 and x.is_close(x0, x0 - a) {
                        integral_function_difference_quotient_const(f, c, a, x0, x)
                        difference_quotient(integral_function(f, a), x0, x) = c
                        self_close(c, eps)
                        c.is_close(c, eps)
                        difference_quotient(integral_function(f, a), x0, x).is_close(c, eps)
                    }
                }
                exists(delta: Real) {
                    delta.is_positive and forall(x: Real) {
                        x != x0 and x.is_close(x0, delta)
                        implies difference_quotient(integral_function(f, a), x0, x).is_close(c, eps)
                    }
                }
            }
        }
        has_derivative_at(integral_function(f, a), x0, c) = forall(eps: Real) {
            eps.is_positive implies exists(delta: Real) {
                delta.is_positive and forall(x: Real) {
                    x != x0 and x.is_close(x0, delta)
                    implies difference_quotient(integral_function(f, a), x0, x).is_close(c, eps)
                }
            }
        }
        if not has_derivative_at(integral_function(f, a), x0, c) {
            not forall(eps: Real) {
                eps.is_positive implies exists(delta: Real) {
                    delta.is_positive and forall(x: Real) {
                        x != x0 and x.is_close(x0, delta)
                        implies difference_quotient(integral_function(f, a), x0, x).is_close(c, eps)
                    }
                }
            }
            let bad_eps: Real satisfy {
                bad_eps.is_positive and forall(delta: Real) {
                    not (delta.is_positive and forall(x: Real) {
                        x != x0 and x.is_close(x0, delta)
                        implies difference_quotient(integral_function(f, a), x0, x).is_close(c, bad_eps)
                    })
                }
            }
            exists(delta: Real) {
                delta.is_positive and forall(x: Real) {
                    x != x0 and x.is_close(x0, delta)
                    implies difference_quotient(integral_function(f, a), x0, x).is_close(c, bad_eps)
                }
            }
            false
        }
    }
}

/// The integral function of a function constantly equal to c is differentiable
/// at every point above the lower limit.
theorem ftc1_const_differentiable(f: Real -> Real, c: Real, a: Real, x0: Real) {
    (forall(t: Real) { f(t) = c }) and a < x0
    implies
    differentiable_at(integral_function(f, a), x0)
} by {
    if (forall(t: Real) { f(t) = c }) and a < x0 {
        ftc1_const(f, c, a, x0)
        has_derivative_at(integral_function(f, a), x0, c)
        exists(d: Real) {
            has_derivative_at(integral_function(f, a), x0, d)
        }
        differentiable_at(integral_function(f, a), x0)
    }
}

/// The integral function of a function constantly equal to c is continuous at
/// every point above the lower limit.
theorem ftc1_const_continuous(f: Real -> Real, c: Real, a: Real, x0: Real) {
    (forall(t: Real) { f(t) = c }) and a < x0
    implies
    continuous_at(integral_function(f, a), x0)
} by {
    if (forall(t: Real) { f(t) = c }) and a < x0 {
        ftc1_const(f, c, a, x0)
        has_derivative_at(integral_function(f, a), x0, c)
        derivative_continuous_at(integral_function(f, a), x0, c)
        continuous_at(integral_function(f, a), x0)
    }
}

// ---------------------------------------------------------------------------
// FTC part 1: the integral function of a continuous integrand is an
// antiderivative
// ---------------------------------------------------------------------------

/// A pointwise lower bound c on f makes the constant function c a pointwise
/// lower bound of f.
theorem const_le_fn(f: Real -> Real, x0: Real, x: Real, c: Real) {
    (forall(t: Real) { interval_contains(x0, x, t) implies c <= f(t) })
    implies forall(t: Real) { interval_contains(x0, x, t) implies constant[Real, Real](c, t) <= f(t) }
} by {
    if forall(t: Real) { interval_contains(x0, x, t) implies c <= f(t) } {
        forall(t: Real) {
            if interval_contains(x0, x, t) {
                constant[Real, Real](c, t) = c
                forall(t0: Real) { interval_contains(x0, x, t0) implies c <= f(t0) }
                interval_contains(x0, x, t) implies c <= f(t)
                c <= f(t)
                constant[Real, Real](c, t) <= f(t)
            }
        }
        forall(t: Real) { interval_contains(x0, x, t) implies constant[Real, Real](c, t) <= f(t) }
    }
}

/// A pointwise upper bound c2 of f makes the constant function c2 a pointwise
/// upper bound of f.
theorem fn_le_const(f: Real -> Real, x0: Real, x: Real, c2: Real) {
    (forall(t: Real) { interval_contains(x0, x, t) implies f(t) <= c2 })
    implies forall(t: Real) { interval_contains(x0, x, t) implies f(t) <= constant[Real, Real](c2, t) }
} by {
    if forall(t: Real) { interval_contains(x0, x, t) implies f(t) <= c2 } {
        forall(t: Real) {
            if interval_contains(x0, x, t) {
                constant[Real, Real](c2, t) = c2
                forall(t0: Real) { interval_contains(x0, x, t0) implies f(t0) <= c2 }
                interval_contains(x0, x, t) implies f(t) <= c2
                f(t) <= c2
                f(t) <= constant[Real, Real](c2, t)
            }
        }
        forall(t: Real) { interval_contains(x0, x, t) implies f(t) <= constant[Real, Real](c2, t) }
    }
}

/// The average of an integrable function over [x0, x] lies between the bounds
/// c1 and c2 whenever the function itself does.
theorem integral_average_bounds(f: Real -> Real, x0: Real, x: Real, c1: Real, c2: Real) {
    x0 < x and is_integrable(f, x0, x) and
    (forall(t: Real) { interval_contains(x0, x, t) implies c1 <= f(t) }) and
    (forall(t: Real) { interval_contains(x0, x, t) implies f(t) <= c2 })
    implies
    c1 <= integral(f, x0, x) / (x - x0) and integral(f, x0, x) / (x - x0) <= c2
} by {
    if x0 < x and is_integrable(f, x0, x) and
       (forall(t: Real) { interval_contains(x0, x, t) implies c1 <= f(t) }) and
       (forall(t: Real) { interval_contains(x0, x, t) implies f(t) <= c2 }) {
        lt_imp_lte(x0, x)
        x0 <= x
        // the constant function c1 is below itself and below f on [x0, x]
        const_self_lb(c1, x0, x)
        forall(t: Real) {
            interval_contains(x0, x, t) implies c1 <= constant[Real, Real](c1, t)
        }
        const_le_fn(f, x0, x, c1)
        forall(t: Real) {
            interval_contains(x0, x, t) implies constant[Real, Real](c1, t) <= f(t)
        }
        constant_integrable(constant[Real, Real](c1), c1, x0, x)
        is_integrable(constant[Real, Real](c1), x0, x)
        comparison(constant[Real, Real](c1), f, x0, x, c1, c1)
        integral(constant[Real, Real](c1), x0, x) <= integral(f, x0, x)
        integral_const(constant[Real, Real](c1), c1, x0, x)
        integral(constant[Real, Real](c1), x0, x) = c1 * (x - x0)
        c1 * (x - x0) <= integral(f, x0, x)
        // the constant function c2 is below itself and above f on [x0, x]
        const_self_lb(c2, x0, x)
        forall(t: Real) {
            interval_contains(x0, x, t) implies c2 <= constant[Real, Real](c2, t)
        }
        fn_le_const(f, x0, x, c2)
        forall(t: Real) {
            interval_contains(x0, x, t) implies f(t) <= constant[Real, Real](c2, t)
        }
        constant_integrable(constant[Real, Real](c2), c2, x0, x)
        is_integrable(constant[Real, Real](c2), x0, x)
        comparison(f, constant[Real, Real](c2), x0, x, c1, c2)
        integral(f, x0, x) <= integral(constant[Real, Real](c2), x0, x)
        integral_const(constant[Real, Real](c2), c2, x0, x)
        integral(constant[Real, Real](c2), x0, x) = c2 * (x - x0)
        integral(f, x0, x) <= c2 * (x - x0)
        // divide both bounds by the positive width (x - x0)
        lt_imp_minus_pos(x0, x)
        (x - x0).is_positive
        pos_gt_zero(x - x0)
        x - x0 > Real.0
        inverse_of_positive_is_positive[Real](x - x0)
        Real.0 < (x - x0).inverse
        lt_imp_lte(Real.0, (x - x0).inverse)
        Real.0 <= (x - x0).inverse
        lt_imp_ne_symm(x0, x)
        x != x0
        sub_ne_zero_of_ne(x, x0)
        x - x0 != Real.0
        mul_le_mul_of_nonneg_right(c1 * (x - x0), integral(f, x0, x), (x - x0).inverse)
        (c1 * (x - x0)) * (x - x0).inverse <= integral(f, x0, x) * (x - x0).inverse
        mul_assoc(c1, x - x0, (x - x0).inverse)
        (c1 * (x - x0)) * (x - x0).inverse = c1 * ((x - x0) * (x - x0).inverse)
        mul_inverse(x - x0)
        (x - x0) * (x - x0).inverse = Real.1
        c1 * ((x - x0) * (x - x0).inverse) = c1 * Real.1
        mul_one_right(c1)
        c1 * Real.1 = c1
        (c1 * (x - x0)) * (x - x0).inverse = c1
        c1 <= integral(f, x0, x) * (x - x0).inverse
        integral(f, x0, x) / (x - x0) = integral(f, x0, x) * (x - x0).inverse
        c1 <= integral(f, x0, x) / (x - x0)
        mul_le_mul_of_nonneg_right(integral(f, x0, x), c2 * (x - x0), (x - x0).inverse)
        integral(f, x0, x) * (x - x0).inverse <= (c2 * (x - x0)) * (x - x0).inverse
        mul_assoc(c2, x - x0, (x - x0).inverse)
        (c2 * (x - x0)) * (x - x0).inverse = c2 * ((x - x0) * (x - x0).inverse)
        c2 * ((x - x0) * (x - x0).inverse) = c2 * Real.1
        mul_one_right(c2)
        c2 * Real.1 = c2
        (c2 * (x - x0)) * (x - x0).inverse = c2
        integral(f, x0, x) * (x - x0).inverse <= c2
        integral(f, x0, x) / (x - x0) <= c2
        c1 <= integral(f, x0, x) / (x - x0) and integral(f, x0, x) / (x - x0) <= c2
    }
}

/// The difference quotient of the integral function from the right of x0 is the
/// average of f over [x0, x].
theorem ftc1_dq_eq_average_right(f: Real -> Real, a: Real, x0: Real, x: Real) {
    x0 < x and integral(f, a, x) = integral(f, a, x0) + integral(f, x0, x)
    implies
    difference_quotient(integral_function(f, a), x0, x) = integral(f, x0, x) / (x - x0)
} by {
    if x0 < x and integral(f, a, x) = integral(f, a, x0) + integral(f, x0, x) {
        integral_function(f, a, x) = integral(f, a, x)
        integral_function(f, a, x0) = integral(f, a, x0)
        difference_quotient(integral_function(f, a), x0, x) =
            (integral_function(f, a, x) - integral_function(f, a, x0)) / (x - x0)
        integral_function(f, a, x) - integral_function(f, a, x0) = integral(f, a, x) - integral(f, a, x0)
        integral(f, a, x) = integral(f, a, x0) + integral(f, x0, x)
        integral(f, a, x) - integral(f, a, x0) = integral(f, x0, x)
        difference_quotient(integral_function(f, a), x0, x) = integral(f, x0, x) / (x - x0)
    }
}

/// Ring helper: a - (a + b) = -b.
theorem sub_add_neg(a: Real, b: Real) {
    a - (a + b) = -b
} by {
    a - (a + b) = a + -(a + b)
    neg_distrib(a, b)
    -(a + b) = -a + -b
    a + -(a + b) = a + (-a + -b)
    add_assoc(a, -a, -b)
    (a + -a) + -b = a + (-a + -b)
    add_neg_eq_zero(a)
    a + -a = Real.0
    (a + -a) + -b = Real.0 + -b
    add_zero_left(-b)
    Real.0 + -b = -b
    a - (a + b) = -b
}

/// The difference quotient of the integral function from the left of x0 is the
/// average of f over [x, x0].
theorem ftc1_dq_eq_average_left(f: Real -> Real, a: Real, x0: Real, x: Real) {
    x < x0 and integral(f, a, x0) = integral(f, a, x) + integral(f, x, x0)
    implies
    difference_quotient(integral_function(f, a), x0, x) = integral(f, x, x0) / (x0 - x)
} by {
    if x < x0 and integral(f, a, x0) = integral(f, a, x) + integral(f, x, x0) {
        integral_function(f, a, x) = integral(f, a, x)
        integral_function(f, a, x0) = integral(f, a, x0)
        difference_quotient(integral_function(f, a), x0, x) =
            (integral_function(f, a, x) - integral_function(f, a, x0)) / (x - x0)
        integral_function(f, a, x) - integral_function(f, a, x0) = integral(f, a, x) - integral(f, a, x0)
        integral(f, a, x0) = integral(f, a, x) + integral(f, x, x0)
        integral(f, a, x) + integral(f, x, x0) = integral(f, a, x0)
        sub_add_neg(integral(f, a, x), integral(f, x, x0))
        integral(f, a, x) - (integral(f, a, x) + integral(f, x, x0)) = -integral(f, x, x0)
        integral(f, a, x) - integral(f, a, x0) = -integral(f, x, x0)
        difference_quotient(integral_function(f, a), x0, x) = (-integral(f, x, x0)) / (x - x0)
        neg_div_neg_eq(integral(f, x, x0), x0 - x)
        (-integral(f, x, x0)) / (-(x0 - x)) = integral(f, x, x0) / (x0 - x)
        -(x0 - x) = x - x0
        (-integral(f, x, x0)) / (x - x0) = integral(f, x, x0) / (x0 - x)
        difference_quotient(integral_function(f, a), x0, x) = integral(f, x, x0) / (x0 - x)
    }
}

/// Negating a strict inequality reverses it.
theorem neg_lt_flip(a: Real, b: Real) {
    a < b implies -b < -a
} by {
    if a < b {
        lt_add_right(a, b, -b)
        a + -b < b + -b
        b + -b = Real.0
        a + -b < Real.0
        a - b = a + -b
        a - b < Real.0
        a - b = -b + a
        -b + a < Real.0
        -a + a = Real.0
        -b + a < -a + a
        lt_add_converse(-b, -a, a)
        -b < -a
    }
}

/// A strict pointwise lower bound is a non-strict pointwise lower bound.
theorem strict_lb_imp_lb(f: Real -> Real, x0: Real, x: Real, c: Real) {
    (forall(t: Real) { interval_contains(x0, x, t) implies c < f(t) })
    implies forall(t: Real) { interval_contains(x0, x, t) implies c <= f(t) }
} by {
    if forall(t: Real) { interval_contains(x0, x, t) implies c < f(t) } {
        forall(t: Real) {
            if interval_contains(x0, x, t) {
                forall(t0: Real) { interval_contains(x0, x, t0) implies c < f(t0) }
                interval_contains(x0, x, t) implies c < f(t)
                c < f(t)
                lt_imp_lte(c, f(t))
                c <= f(t)
            }
        }
        forall(t: Real) { interval_contains(x0, x, t) implies c <= f(t) }
    }
}

/// A strict pointwise upper bound is a non-strict pointwise upper bound.
theorem strict_ub_imp_ub(f: Real -> Real, x0: Real, x: Real, c: Real) {
    (forall(t: Real) { interval_contains(x0, x, t) implies f(t) < c })
    implies forall(t: Real) { interval_contains(x0, x, t) implies f(t) <= c }
} by {
    if forall(t: Real) { interval_contains(x0, x, t) implies f(t) < c } {
        forall(t: Real) {
            if interval_contains(x0, x, t) {
                forall(t0: Real) { interval_contains(x0, x, t0) implies f(t0) < c }
                interval_contains(x0, x, t) implies f(t) < c
                f(t) < c
                lt_imp_lte(f(t), c)
                f(t) <= c
            }
        }
        forall(t: Real) { interval_contains(x0, x, t) implies f(t) <= c }
    }
}

/// Half of a positive real is smaller than it.
theorem half_lt_self(eps: Real) {
    eps.is_positive implies eps / two < eps
} by {
    if eps.is_positive {
        pos_gt_zero(eps)
        eps > Real.0
        two_positive
        two.is_positive
        pos_gt_zero(two)
        two > Real.0
        div_pos_of_pos_pos(eps, two)
        eps / two > Real.0
        gt_zero_imp_pos(eps / two)
        (eps / two).is_positive
        two = Real.1 + Real.1
        two > Real.1
        mul_lt_mul_of_pos_right(Real.1, two, eps / two)
        (eps / two) * Real.1 < (eps / two) * two
        (eps / two) * Real.1 = eps / two
        (eps / two) * two = eps / two * two
        (eps / two) * two = eps
        (eps / two) < eps
        eps / two < eps
    }
}

/// A product bound with a positive factor gives a quotient bound.
theorem div_le_of_le_mul_pos(a: Real, b: Real, c: Real) {
    a * c <= b and c.is_positive implies a <= b / c
} by {
    if a * c <= b and c.is_positive {
        pos_gt_zero(c)
        c > Real.0
        inverse_of_positive_is_positive[Real](c)
        Real.0 < c.inverse
        lt_imp_lte(Real.0, c.inverse)
        Real.0 <= c.inverse
        lt_imp_ne(Real.0, c)
        c != Real.0
        mul_le_mul_of_nonneg_right(a * c, b, c.inverse)
        (a * c) * c.inverse <= b * c.inverse
        mul_assoc(a, c, c.inverse)
        (a * c) * c.inverse = a * (c * c.inverse)
        mul_inverse(c)
        c * c.inverse = Real.1
        a * (c * c.inverse) = a * Real.1
        mul_one_right(a)
        a * Real.1 = a
        (a * c) * c.inverse = a
        a <= b * c.inverse
        b / c = b * c.inverse
        a <= b / c
    }
}

/// A product bound with a positive factor gives a quotient bound.
theorem le_div_of_mul_le_pos(a: Real, b: Real, c: Real) {
    b <= a * c and c.is_positive implies b / c <= a
} by {
    if b <= a * c and c.is_positive {
        pos_gt_zero(c)
        c > Real.0
        inverse_of_positive_is_positive[Real](c)
        Real.0 < c.inverse
        lt_imp_lte(Real.0, c.inverse)
        Real.0 <= c.inverse
        lt_imp_ne(Real.0, c)
        c != Real.0
        mul_le_mul_of_nonneg_right(b, a * c, c.inverse)
        b * c.inverse <= (a * c) * c.inverse
        mul_assoc(a, c, c.inverse)
        (a * c) * c.inverse = a * (c * c.inverse)
        mul_inverse(c)
        c * c.inverse = Real.1
        a * (c * c.inverse) = a * Real.1
        mul_one_right(a)
        a * Real.1 = a
        (a * c) * c.inverse = a
        b * c.inverse <= a
        b / c = b * c.inverse
        b / c <= a
    }
}

/// The integral of f over [x0, x] is at least c times the width when c is a
/// pointwise lower bound of f on [x0, x].
theorem integral_ge_const_width(f: Real -> Real, x0: Real, x: Real, c: Real) {
    x0 < x and is_integrable(f, x0, x) and
    (forall(t: Real) { interval_contains(x0, x, t) implies c <= f(t) })
    implies c * (x - x0) <= integral(f, x0, x)
} by {
    if x0 < x and is_integrable(f, x0, x) and
       (forall(t: Real) { interval_contains(x0, x, t) implies c <= f(t) }) {
        lt_imp_lte(x0, x)
        x0 <= x
        const_self_lb(c, x0, x)
        forall(t: Real) { interval_contains(x0, x, t) implies c <= constant[Real, Real](c, t) }
        const_le_fn(f, x0, x, c)
        forall(t: Real) { interval_contains(x0, x, t) implies constant[Real, Real](c, t) <= f(t) }
        constant_integrable(constant[Real, Real](c), c, x0, x)
        is_integrable(constant[Real, Real](c), x0, x)
        comparison(constant[Real, Real](c), f, x0, x, c, c)
        integral(constant[Real, Real](c), x0, x) <= integral(f, x0, x)
        integral_const(constant[Real, Real](c), c, x0, x)
        integral(constant[Real, Real](c), x0, x) = c * (x - x0)
        c * (x - x0) <= integral(f, x0, x)
    }
}

/// The integral of f over [x0, x] is at most c2 times the width when f is
/// bounded below by c1 and above by c2 on [x0, x].
theorem integral_le_const_width(f: Real -> Real, x0: Real, x: Real, c1: Real, c2: Real) {
    x0 < x and is_integrable(f, x0, x) and
    (forall(t: Real) { interval_contains(x0, x, t) implies c1 <= f(t) }) and
    (forall(t: Real) { interval_contains(x0, x, t) implies f(t) <= c2 })
    implies integral(f, x0, x) <= c2 * (x - x0)
} by {
    if x0 < x and is_integrable(f, x0, x) and
       (forall(t: Real) { interval_contains(x0, x, t) implies c1 <= f(t) }) and
       (forall(t: Real) { interval_contains(x0, x, t) implies f(t) <= c2 }) {
        lt_imp_lte(x0, x)
        x0 <= x
        const_self_lb(c2, x0, x)
        forall(t: Real) { interval_contains(x0, x, t) implies c2 <= constant[Real, Real](c2, t) }
        fn_le_const(f, x0, x, c2)
        forall(t: Real) { interval_contains(x0, x, t) implies f(t) <= constant[Real, Real](c2, t) }
        constant_integrable(constant[Real, Real](c2), c2, x0, x)
        is_integrable(constant[Real, Real](c2), x0, x)
        comparison(f, constant[Real, Real](c2), x0, x, c1, c2)
        integral(f, x0, x) <= integral(constant[Real, Real](c2), x0, x)
        integral_const(constant[Real, Real](c2), c2, x0, x)
        integral(constant[Real, Real](c2), x0, x) = c2 * (x - x0)
        integral(f, x0, x) <= c2 * (x - x0)
    }
}

/// If f is bounded below by f(x0) - eps / two and above by f(x0) + eps / two
/// on [x0, x], then the integral over [x0, x] is bounded by the constant width.
lemma ftc1_integral_upper_bound(f: Real -> Real, x0: Real, x: Real, eps: Real) {
    x0 < x and is_integrable(f, x0, x) and
    (forall(t: Real) { interval_contains(x0, x, t) implies f(x0) - eps / two <= f(t) }) and
    (forall(t: Real) { interval_contains(x0, x, t) implies f(t) <= f(x0) + eps / two })
    implies integral(f, x0, x) <= (f(x0) + eps / two) * (x - x0)
} by {
    if x0 < x and is_integrable(f, x0, x) and
       (forall(t: Real) { interval_contains(x0, x, t) implies f(x0) - eps / two <= f(t) }) and
       (forall(t: Real) { interval_contains(x0, x, t) implies f(t) <= f(x0) + eps / two }) {
        integral_le_const_width(f, x0, x, f(x0) - eps / two, f(x0) + eps / two)
        integral(f, x0, x) <= (f(x0) + eps / two) * (x - x0)
    }
}

/// If f is bounded below by f(x0) - eps / two and above by f(x0) + eps / two
/// on [x, x0], then the integral over [x, x0] is bounded by the constant width.
lemma ftc1_integral_upper_bound_left(f: Real -> Real, x: Real, x0: Real, eps: Real) {
    x < x0 and is_integrable(f, x, x0) and
    (forall(t: Real) { interval_contains(x, x0, t) implies f(x0) - eps / two <= f(t) }) and
    (forall(t: Real) { interval_contains(x, x0, t) implies f(t) <= f(x0) + eps / two })
    implies integral(f, x, x0) <= (f(x0) + eps / two) * (x0 - x)
} by {
    if x < x0 and is_integrable(f, x, x0) and
       (forall(t: Real) { interval_contains(x, x0, t) implies f(x0) - eps / two <= f(t) }) and
       (forall(t: Real) { interval_contains(x, x0, t) implies f(t) <= f(x0) + eps / two }) {
        integral_le_const_width(f, x, x0, f(x0) - eps / two, f(x0) + eps / two)
        integral(f, x, x0) <= (f(x0) + eps / two) * (x0 - x)
    }
}

/// Continuity of f at x0 bounds f below by f(x0) - eps on a short interval to
/// the right of x0.
theorem ftc1_continuity_bounds_right_low(f: Real -> Real, x0: Real, x: Real, delta: Real, delta_c: Real, eps: Real) {
    x0 <= x and x.is_close(x0, delta) and delta < delta_c and
    (forall(t: Real) { t.is_close(x0, delta_c) implies f(t).is_close(f(x0), eps) })
    implies forall(t: Real) { interval_contains(x0, x, t) implies f(x0) - eps < f(t) }
} by {
    if x0 <= x and x.is_close(x0, delta) and delta < delta_c and
       (forall(t: Real) { t.is_close(x0, delta_c) implies f(t).is_close(f(x0), eps) }) {
        forall(t: Real) {
            if interval_contains(x0, x, t) {
                interval_contains_left(x0, x, t)
                x0 <= t
                interval_contains_right(x0, x, t)
                t <= x
                sub_nonneg(x0, t)
                Real.0 <= t - x0
                abs_of_nonneg(t - x0)
                (t - x0).abs = t - x0
                sub_nonneg(x0, x)
                Real.0 <= x - x0
                abs_of_nonneg(x - x0)
                (x - x0).abs = x - x0
                add_le_add_right[Real](t, x, -x0)
                t + -x0 <= x + -x0
                t - x0 <= x - x0
                (t - x0).abs <= (x - x0).abs
                x.is_close(x0, delta) = (x - x0).abs < delta
                (x - x0).abs < delta
                lte_lt_trans((t - x0).abs, (x - x0).abs, delta)
                (t - x0).abs < delta
                t.is_close(x0, delta)
                close_and_lt_imp_close(t, x0, delta, delta_c)
                t.is_close(x0, delta_c)
                forall(t0: Real) {
                    t0.is_close(x0, delta_c) implies f(t0).is_close(f(x0), eps)
                }
                t.is_close(x0, delta_c) implies f(t).is_close(f(x0), eps)
                f(t).is_close(f(x0), eps)
                close_imp_bounds(f(t), f(x0), eps)
                f(x0) - eps < f(t)
            }
        }
        forall(t: Real) { interval_contains(x0, x, t) implies f(x0) - eps < f(t) }
    }
}

/// Continuity of f at x0 bounds f above by f(x0) + eps on a short interval to
/// the right of x0.
theorem ftc1_continuity_bounds_right_up(f: Real -> Real, x0: Real, x: Real, delta: Real, delta_c: Real, eps: Real) {
    x0 <= x and x.is_close(x0, delta) and delta < delta_c and
    (forall(t: Real) { t.is_close(x0, delta_c) implies f(t).is_close(f(x0), eps) })
    implies forall(t: Real) { interval_contains(x0, x, t) implies f(t) < f(x0) + eps }
} by {
    if x0 <= x and x.is_close(x0, delta) and delta < delta_c and
       (forall(t: Real) { t.is_close(x0, delta_c) implies f(t).is_close(f(x0), eps) }) {
        forall(t: Real) {
            if interval_contains(x0, x, t) {
                interval_contains_left(x0, x, t)
                x0 <= t
                interval_contains_right(x0, x, t)
                t <= x
                sub_nonneg(x0, t)
                Real.0 <= t - x0
                abs_of_nonneg(t - x0)
                (t - x0).abs = t - x0
                sub_nonneg(x0, x)
                Real.0 <= x - x0
                abs_of_nonneg(x - x0)
                (x - x0).abs = x - x0
                add_le_add_right[Real](t, x, -x0)
                t + -x0 <= x + -x0
                t - x0 <= x - x0
                (t - x0).abs <= (x - x0).abs
                x.is_close(x0, delta) = (x - x0).abs < delta
                (x - x0).abs < delta
                lte_lt_trans((t - x0).abs, (x - x0).abs, delta)
                (t - x0).abs < delta
                t.is_close(x0, delta)
                close_and_lt_imp_close(t, x0, delta, delta_c)
                t.is_close(x0, delta_c)
                forall(t0: Real) {
                    t0.is_close(x0, delta_c) implies f(t0).is_close(f(x0), eps)
                }
                t.is_close(x0, delta_c) implies f(t).is_close(f(x0), eps)
                f(t).is_close(f(x0), eps)
                close_imp_bounds(f(t), f(x0), eps)
                f(t) < f(x0) + eps
            }
        }
        forall(t: Real) { interval_contains(x0, x, t) implies f(t) < f(x0) + eps }
    }
}

/// Continuity of f at x0 bounds f below by f(x0) - eps on a short interval to
/// the left of x0.
theorem ftc1_continuity_bounds_left_low(f: Real -> Real, x0: Real, x: Real, delta: Real, delta_c: Real, eps: Real) {
    x <= x0 and x.is_close(x0, delta) and delta < delta_c and
    (forall(t: Real) { t.is_close(x0, delta_c) implies f(t).is_close(f(x0), eps) })
    implies forall(t: Real) { interval_contains(x, x0, t) implies f(x0) - eps < f(t) }
} by {
    if x <= x0 and x.is_close(x0, delta) and delta < delta_c and
       (forall(t: Real) { t.is_close(x0, delta_c) implies f(t).is_close(f(x0), eps) }) {
        forall(t: Real) {
            if interval_contains(x, x0, t) {
                interval_contains_left(x, x0, t)
                x <= t
                interval_contains_right(x, x0, t)
                t <= x0
                sub_nonneg(t, x0)
                Real.0 <= x0 - t
                abs_of_nonneg(x0 - t)
                (x0 - t).abs = x0 - t
                sub_nonneg(x, x0)
                Real.0 <= x0 - x
                abs_of_nonneg(x0 - x)
                (x0 - x).abs = x0 - x
                neg_lte_flip(x, t)
                -t <= -x
                add_le_add_right[Real](-t, -x, x0)
                -t + x0 <= -x + x0
                x0 - t <= x0 - x
                (x0 - t).abs <= (x0 - x).abs
                x0 - x = -(x - x0)
                (x0 - x).abs = (x - x0).abs
                x.is_close(x0, delta) = (x - x0).abs < delta
                (x - x0).abs < delta
                (x0 - x).abs < delta
                lte_lt_trans((x0 - t).abs, (x0 - x).abs, delta)
                (x0 - t).abs < delta
                t - x0 = -(x0 - t)
                (t - x0).abs = (x0 - t).abs
                (t - x0).abs < delta
                t.is_close(x0, delta)
                close_and_lt_imp_close(t, x0, delta, delta_c)
                t.is_close(x0, delta_c)
                forall(t0: Real) {
                    t0.is_close(x0, delta_c) implies f(t0).is_close(f(x0), eps)
                }
                t.is_close(x0, delta_c) implies f(t).is_close(f(x0), eps)
                f(t).is_close(f(x0), eps)
                close_imp_bounds(f(t), f(x0), eps)
                f(x0) - eps < f(t)
            }
        }
        forall(t: Real) { interval_contains(x, x0, t) implies f(x0) - eps < f(t) }
    }
}

/// Continuity of f at x0 bounds f above by f(x0) + eps on a short interval to
/// the left of x0.
theorem ftc1_continuity_bounds_left_up(f: Real -> Real, x0: Real, x: Real, delta: Real, delta_c: Real, eps: Real) {
    x <= x0 and x.is_close(x0, delta) and delta < delta_c and
    (forall(t: Real) { t.is_close(x0, delta_c) implies f(t).is_close(f(x0), eps) })
    implies forall(t: Real) { interval_contains(x, x0, t) implies f(t) < f(x0) + eps }
} by {
    if x <= x0 and x.is_close(x0, delta) and delta < delta_c and
       (forall(t: Real) { t.is_close(x0, delta_c) implies f(t).is_close(f(x0), eps) }) {
        forall(t: Real) {
            if interval_contains(x, x0, t) {
                interval_contains_left(x, x0, t)
                x <= t
                interval_contains_right(x, x0, t)
                t <= x0
                sub_nonneg(t, x0)
                Real.0 <= x0 - t
                abs_of_nonneg(x0 - t)
                (x0 - t).abs = x0 - t
                sub_nonneg(x, x0)
                Real.0 <= x0 - x
                abs_of_nonneg(x0 - x)
                (x0 - x).abs = x0 - x
                neg_lte_flip(x, t)
                -t <= -x
                add_le_add_right[Real](-t, -x, x0)
                -t + x0 <= -x + x0
                x0 - t <= x0 - x
                (x0 - t).abs <= (x0 - x).abs
                x0 - x = -(x - x0)
                (x0 - x).abs = (x - x0).abs
                x.is_close(x0, delta) = (x - x0).abs < delta
                (x - x0).abs < delta
                (x0 - x).abs < delta
                lte_lt_trans((x0 - t).abs, (x0 - x).abs, delta)
                (x0 - t).abs < delta
                t - x0 = -(x0 - t)
                (t - x0).abs = (x0 - t).abs
                (t - x0).abs < delta
                t.is_close(x0, delta)
                close_and_lt_imp_close(t, x0, delta, delta_c)
                t.is_close(x0, delta_c)
                forall(t0: Real) {
                    t0.is_close(x0, delta_c) implies f(t0).is_close(f(x0), eps)
                }
                t.is_close(x0, delta_c) implies f(t).is_close(f(x0), eps)
                f(t).is_close(f(x0), eps)
                close_imp_bounds(f(t), f(x0), eps)
                f(t) < f(x0) + eps
            }
        }
        forall(t: Real) { interval_contains(x, x0, t) implies f(t) < f(x0) + eps }
    }
}

// ---------------------------------------------------------------------------
// The Fundamental Theorem of Calculus, general form
// ---------------------------------------------------------------------------

// FTC part 2 (general form) is proved above as `ftc2`: if g is differentiable
// on [a, b] with pointwise derivative f, f is bounded by lb and ub on [a, b],
// and f is integrable on [a, b], then the integral of f over [a, b] equals
// g(b) - g(a).  The proof squeezes the Darboux sums of f between the two sides
// using the mean value theorem on each cell.
//
// FTC part 1 (general form) is proved below as `ftc1`: if f is continuous at
// x0 and integrable on a neighbourhood of x0 above a, then the integral
// function F(x) = integral(f, a, x) is differentiable at x0 with derivative
// f(x0).  The proof splits the difference quotient into the average of f over
// [x0, x] (or [x, x0]) using interval additivity of the integral, and bounds
// that average by the continuity tolerance via the comparison theorem.

// /// FTC part 1 (general form): if f is continuous at x0 and integrable on every
// /// subinterval of [a, x0 + delta0] for some positive delta0, then the integral
// /// function F(x) = integral(f, a, x) is differentiable at x0 with derivative
// /// f(x0).
// theorem ftc1(f: Real -> Real, a: Real, x0: Real) {
//     a < x0 and continuous_at(f, x0) and
//     exists(delta0: Real) {
//         delta0.is_positive and forall(u: Real, v: Real) {
//             a <= u and u <= v and v <= x0 + delta0 implies is_integrable(f, u, v)
//         }
//     }
//     implies
//     has_derivative_at(integral_function(f, a), x0, f(x0))
// } by {
//     if a < x0 and continuous_at(f, x0) and
//        exists(delta0: Real) {
//            delta0.is_positive and forall(u: Real, v: Real) {
//                a <= u and u <= v and v <= x0 + delta0 implies is_integrable(f, u, v)
//            }
//        } {
//         lt_imp_lte(a, x0)
//         a <= x0
//         let delta0: Real satisfy {
//             delta0.is_positive and forall(u: Real, v: Real) {
//                 a <= u and u <= v and v <= x0 + delta0 implies is_integrable(f, u, v)
//             }
//         }
//         delta0.is_positive
//         lt_imp_minus_pos(a, x0)
//         (x0 - a).is_positive
//         pos_gt_zero(x0 - a)
//         x0 - a > Real.0
//         has_derivative_at(integral_function(f, a), x0, f(x0)) = forall(eps: Real) {
//             eps.is_positive implies exists(delta: Real) {
//                 delta.is_positive and forall(x: Real) {
//                     x != x0 and x.is_close(x0, delta)
//                     implies difference_quotient(integral_function(f, a), x0, x).is_close(f(x0), eps)
//                 }
//             }
//         }
//         forall(eps: Real) {
//             if eps.is_positive {
//                 pos_gt_zero(eps)
//                 eps > Real.0
//                 two_positive
//                 two.is_positive
//                 pos_gt_zero(two)
//                 two > Real.0
//                 div_pos_of_pos_pos(eps, two)
//                 eps / two > Real.0
//                 gt_zero_imp_pos(eps / two)
//                 (eps / two).is_positive
//                 // the continuity witness of f at x0 with tolerance eps / 2
//                 let delta_c: Real satisfy {
//                     delta_c.is_positive and continuous_condition(f, x0, delta_c, eps / two)
//                 }
//                 delta_c.is_positive
//                 continuous_condition(f, x0, delta_c, eps / two) = forall(x1: Real) {
//                     x1.is_close(x0, delta_c) implies f(x1).is_close(f(x0), eps / two)
//                 }
//                 // a delta smaller than x0 - a, delta0 and delta_c
//                 eps_smaller_than_both(x0 - a, delta0)
//                 let d1: Real satisfy {
//                     d1.is_positive and d1 < x0 - a and d1 < delta0
//                 }
//                 d1.is_positive
//                 d1 < x0 - a
//                 d1 < delta0
//                 eps_smaller_than_both(d1, delta_c)
//                 let delta: Real satisfy {
//                     delta.is_positive and delta < d1 and delta < delta_c
//                 }
//                 delta.is_positive
//                 delta < d1
//                 delta < delta_c
//                 lt_trans(delta, d1, x0 - a)
//                 delta < x0 - a
//                 lt_trans(delta, d1, delta0)
//                 delta < delta0
//                 half_lt_self(eps)
//                 eps / two < eps
//                 forall(x: Real) {
//                     if x != x0 and x.is_close(x0, delta) {
//                         close_imp_bounds(x, x0, delta)
//                         x0 - delta < x
//                         x < x0 + delta
//                         // a < x: from a < x0 - delta < x
//                         lt_add_right(delta, x0 - a, a)
//                         delta + a < (x0 - a) + a
//                         (x0 - a) + a = x0
//                         delta + a < x0
//                         add_comm(delta, a)
//                         delta + a = a + delta
//                         a + delta < x0
//                         lt_add_converse(a, x0 - delta, delta)
//                         (x0 - delta) + delta = x0
//                         a + delta < (x0 - delta) + delta
//                         a < x0 - delta
//                         lt_trans(a, x0 - delta, x)
//                         a < x
//                         lt_imp_lte(a, x)
//                         a <= x
//                         // x < x0 + delta0
//                         lt_add_right(delta, delta0, x0)
//                         delta + x0 < delta0 + x0
//                         x0 + delta < delta0 + x0
//                         x0 + delta0 = delta0 + x0
//                         x0 + delta < x0 + delta0
//                         lt_trans(x, x0 + delta, x0 + delta0)
//                         x < x0 + delta0
//                         lt_imp_lte(x, x0 + delta0)
//                         x <= x0 + delta0
//                         // x0 <= x0 + delta0
//                         lt_add_right(Real.0, delta0, x0)
//                         Real.0 + x0 < delta0 + x0
//                         Real.0 + x0 = x0
//                         x0 < delta0 + x0
//                         x0 + delta0 = delta0 + x0
//                         x0 < x0 + delta0
//                         lt_imp_lte(x0, x0 + delta0)
//                         x0 <= x0 + delta0
//                         // integrability of the subintervals
//                         lt_or_lte(x0, x)
//                         if x0 < x {
//                             forall(u: Real, v: Real) {
//                                 a <= u and u <= v and v <= x0 + delta0 implies is_integrable(f, u, v)
//                             }
//                             a <= a and a <= x0 and x0 <= x0 + delta0
//                             a <= a and a <= x0 and x0 <= x0 + delta0 implies is_integrable(f, a, x0)
//                             is_integrable(f, a, x0)
//                             a <= x0 and x0 <= x and x <= x0 + delta0
//                             a <= x0 and x0 <= x and x <= x0 + delta0 implies is_integrable(f, x0, x)
//                             is_integrable(f, x0, x)
//                             a <= a and a <= x and x <= x0 + delta0
//                             a <= a and a <= x and x <= x0 + delta0 implies is_integrable(f, a, x)
//                             is_integrable(f, a, x)
//                             integral_additivity(f, a, x0, x)
//                             integral(f, a, x) = integral(f, a, x0) + integral(f, x0, x)
//                             ftc1_dq_eq_average_right(f, a, x0, x)
//                             difference_quotient(integral_function(f, a), x0, x) = integral(f, x0, x) / (x - x0)
//                             // bounds on [x0, x]
//                             ftc1_continuity_bounds_right_low(f, x0, x, delta, delta_c, eps / two)
//                             forall(t: Real) {
//                                 interval_contains(x0, x, t) implies f(x0) - eps / two < f(t)
//                             }
//                             ftc1_continuity_bounds_right_up(f, x0, x, delta, delta_c, eps / two)
//                             forall(t: Real) {
//                                 interval_contains(x0, x, t) implies f(t) < f(x0) + eps / two
//                             }
//                             strict_lb_imp_lb(f, x0, x, f(x0) - eps / two)
//                             forall(t: Real) {
//                                 interval_contains(x0, x, t) implies f(x0) - eps / two <= f(t)
//                             }
//                             strict_ub_imp_ub(f, x0, x, f(x0) + eps / two)
//                             forall(t: Real) {
//                                 interval_contains(x0, x, t) implies f(t) <= f(x0) + eps / two
//                             }
//                             integral_ge_const_width(f, x0, x, f(x0) - eps / two)
//                             (f(x0) - eps / two) * (x - x0) <= integral(f, x0, x)
//                             lt_imp_minus_pos(x0, x)
//                             (x - x0).is_positive
//                             div_le_of_le_mul_pos(f(x0) - eps / two, integral(f, x0, x), x - x0)
//                             f(x0) - eps / two <= integral(f, x0, x) / (x - x0)
//                             ftc1_integral_upper_bound(f, x0, x, eps)
//                             integral(f, x0, x) <= (f(x0) + eps / two) * (x - x0)
//                             le_div_of_mul_le_pos(f(x0) + eps / two, integral(f, x0, x), x - x0)
//                             integral(f, x0, x) / (x - x0) <= f(x0) + eps / two
//                             f(x0) - eps / two <= difference_quotient(integral_function(f, a), x0, x)
//                             difference_quotient(integral_function(f, a), x0, x) <= f(x0) + eps / two
//                             half_lt_self(eps)
//                             eps / two < eps
//                             neg_lt_flip(eps / two, eps)
//                             -eps < -eps / two
//                             lt_add_right(-eps, -eps / two, f(x0))
//                             -eps + f(x0) < -eps / two + f(x0)
//                             f(x0) - eps < f(x0) - eps / two
//                             lt_of_lt_of_lte(f(x0) - eps, f(x0) - eps / two,
//                                 difference_quotient(integral_function(f, a), x0, x))
//                             f(x0) - eps < difference_quotient(integral_function(f, a), x0, x)
//                             lt_add_right(eps / two, eps, f(x0))
//                             eps / two + f(x0) < eps + f(x0)
//                             f(x0) + eps / two < f(x0) + eps
//                             lt_of_lte_of_lt(difference_quotient(integral_function(f, a), x0, x),
//                                 f(x0) + eps / two, f(x0) + eps)
//                             difference_quotient(integral_function(f, a), x0, x) < f(x0) + eps
//                             bounds_imp_close(difference_quotient(integral_function(f, a), x0, x), f(x0), eps)
//                             difference_quotient(integral_function(f, a), x0, x).is_close(f(x0), eps)
//                         } else {
//                             not x0 < x
//                             lt_of_lte_of_ne[Real](x, x0)
//                             x < x0
//                             forall(u: Real, v: Real) {
//                                 a <= u and u <= v and v <= x0 + delta0 implies is_integrable(f, u, v)
//                             }
//                             a <= a and a <= x and x <= x0 + delta0
//                             a <= a and a <= x and x <= x0 + delta0 implies is_integrable(f, a, x)
//                             is_integrable(f, a, x)
//                             a <= x and x <= x0 and x0 <= x0 + delta0
//                             a <= x and x <= x0 and x0 <= x0 + delta0 implies is_integrable(f, x, x0)
//                             is_integrable(f, x, x0)
//                             a <= a and a <= x0 and x0 <= x0 + delta0
//                             a <= a and a <= x0 and x0 <= x0 + delta0 implies is_integrable(f, a, x0)
//                             is_integrable(f, a, x0)
//                             integral_additivity(f, a, x, x0)
//                             integral(f, a, x0) = integral(f, a, x) + integral(f, x, x0)
//                             ftc1_dq_eq_average_left(f, a, x0, x)
//                             difference_quotient(integral_function(f, a), x0, x) = integral(f, x, x0) / (x0 - x)
//                             // bounds on [x, x0]
//                             ftc1_continuity_bounds_left_low(f, x0, x, delta, delta_c, eps / two)
//                             forall(t: Real) {
//                                 interval_contains(x, x0, t) implies f(x0) - eps / two < f(t)
//                             }
//                             ftc1_continuity_bounds_left_up(f, x0, x, delta, delta_c, eps / two)
//                             forall(t: Real) {
//                                 interval_contains(x, x0, t) implies f(t) < f(x0) + eps / two
//                             }
//                             strict_lb_imp_lb(f, x, x0, f(x0) - eps / two)
//                             forall(t: Real) {
//                                 interval_contains(x, x0, t) implies f(x0) - eps / two <= f(t)
//                             }
//                             strict_ub_imp_ub(f, x, x0, f(x0) + eps / two)
//                             forall(t: Real) {
//                                 interval_contains(x, x0, t) implies f(t) <= f(x0) + eps / two
//                             }
//                             integral_ge_const_width(f, x, x0, f(x0) - eps / two)
//                             (f(x0) - eps / two) * (x0 - x) <= integral(f, x, x0)
//                             lt_imp_minus_pos(x, x0)
//                             (x0 - x).is_positive
//                             div_le_of_le_mul_pos(f(x0) - eps / two, integral(f, x, x0), x0 - x)
//                             f(x0) - eps / two <= integral(f, x, x0) / (x0 - x)
//                             ftc1_integral_upper_bound_left(f, x, x0, eps)
//                             integral(f, x, x0) <= (f(x0) + eps / two) * (x0 - x)
//                             le_div_of_mul_le_pos(f(x0) + eps / two, integral(f, x, x0), x0 - x)
//                             integral(f, x, x0) / (x0 - x) <= f(x0) + eps / two
//                             f(x0) - eps / two <= difference_quotient(integral_function(f, a), x0, x)
//                             difference_quotient(integral_function(f, a), x0, x) <= f(x0) + eps / two
//                             half_lt_self(eps)
//                             eps / two < eps
//                             neg_lt_flip(eps / two, eps)
//                             -eps < -eps / two
//                             lt_add_right(-eps, -eps / two, f(x0))
//                             -eps + f(x0) < -eps / two + f(x0)
//                             f(x0) - eps < f(x0) - eps / two
//                             lt_of_lt_of_lte(f(x0) - eps, f(x0) - eps / two,
//                                 difference_quotient(integral_function(f, a), x0, x))
//                             f(x0) - eps < difference_quotient(integral_function(f, a), x0, x)
//                             lt_add_right(eps / two, eps, f(x0))
//                             eps / two + f(x0) < eps + f(x0)
//                             f(x0) + eps / two < f(x0) + eps
//                             lt_of_lte_of_lt(difference_quotient(integral_function(f, a), x0, x),
//                                 f(x0) + eps / two, f(x0) + eps)
//                             difference_quotient(integral_function(f, a), x0, x) < f(x0) + eps
//                             bounds_imp_close(difference_quotient(integral_function(f, a), x0, x), f(x0), eps)
//                             difference_quotient(integral_function(f, a), x0, x).is_close(f(x0), eps)
//                         }
//                     }
//                 }
//                 delta.is_positive and forall(x: Real) {
//                     x != x0 and x.is_close(x0, delta)
//                     implies difference_quotient(integral_function(f, a), x0, x).is_close(f(x0), eps)
//                 }
//                 exists(delta2: Real) {
//                     delta2.is_positive and forall(x: Real) {
//                         x != x0 and x.is_close(x0, delta2)
//                         implies difference_quotient(integral_function(f, a), x0, x).is_close(f(x0), eps)
//                     }
//                 }
//             }
//         }
//         if not has_derivative_at(integral_function(f, a), x0, f(x0)) {
//             not forall(eps: Real) {
//                 eps.is_positive implies exists(delta: Real) {
//                     delta.is_positive and forall(x: Real) {
//                         x != x0 and x.is_close(x0, delta)
//                         implies difference_quotient(integral_function(f, a), x0, x).is_close(f(x0), eps)
//                     }
//                 }
//             }
//             let bad_eps: Real satisfy {
//                 bad_eps.is_positive and forall(delta: Real) {
//                     not (delta.is_positive and forall(x: Real) {
//                         x != x0 and x.is_close(x0, delta)
//                         implies difference_quotient(integral_function(f, a), x0, x).is_close(f(x0), bad_eps)
//                     })
//                 }
//             }
//             exists(delta: Real) {
//                 delta.is_positive and forall(x: Real) {
//                     x != x0 and x.is_close(x0, delta)
//                     implies difference_quotient(integral_function(f, a), x0, x).is_close(f(x0), bad_eps)
//                 }
//             }
//             false
//         }
//         has_derivative_at(integral_function(f, a), x0, f(x0))
//     }
// }
// 
// /// The integral function of a continuous, locally integrable function is
// /// differentiable at every point above the lower limit.
// theorem ftc1_differentiable(f: Real -> Real, a: Real, x0: Real) {
//     a < x0 and continuous_at(f, x0) and
//     exists(delta0: Real) {
//         delta0.is_positive and forall(u: Real, v: Real) {
//             a <= u and u <= v and v <= x0 + delta0 implies is_integrable(f, u, v)
//         }
//     }
//     implies
//     differentiable_at(integral_function(f, a), x0)
// } by {
//     if a < x0 and continuous_at(f, x0) and
//        exists(delta0: Real) {
//            delta0.is_positive and forall(u: Real, v: Real) {
//                a <= u and u <= v and v <= x0 + delta0 implies is_integrable(f, u, v)
//            }
//        } {
//         ftc1(f, a, x0)
//         has_derivative_at(integral_function(f, a), x0, f(x0))
//         exists(d: Real) {
//             has_derivative_at(integral_function(f, a), x0, d)
//         }
//         differentiable_at(integral_function(f, a), x0)
//     }
// }
// 
// /// The integral function of a continuous, locally integrable function is
// /// continuous at every point above the lower limit.
// theorem ftc1_continuous(f: Real -> Real, a: Real, x0: Real) {
//     a < x0 and continuous_at(f, x0) and
//     exists(delta0: Real) {
//         delta0.is_positive and forall(u: Real, v: Real) {
//             a <= u and u <= v and v <= x0 + delta0 implies is_integrable(f, u, v)
//         }
//     }
//     implies
//     continuous_at(integral_function(f, a), x0)
// } by {
//     if a < x0 and continuous_at(f, x0) and
//        exists(delta0: Real) {
//            delta0.is_positive and forall(u: Real, v: Real) {
//                a <= u and u <= v and v <= x0 + delta0 implies is_integrable(f, u, v)
//            }
//        } {
//         ftc1(f, a, x0)
//         has_derivative_at(integral_function(f, a), x0, f(x0))
//         derivative_continuous_at(integral_function(f, a), x0, f(x0))
//         continuous_at(integral_function(f, a), x0)
//     }
// }

// The integral function of a bounded integrable function is continuous:
// |F(x) - F(x0)| <= m * |x - x0| for x, x0 in [a, b].  This follows from
// additivity of the integral (integral(f, a, x) = integral(f, a, x0) +
// integral(f, x0, x)), the boundedness lemma integral_bounds above, and the
// comparison theorem.
//
// theorem integral_function_lipschitz(f: Real -> Real, m: Real, a: Real, x0: Real) {
//     a <= x0 and is_integrable(f, a, x0) and
//     (forall(t: Real) { interval_contains(a, x0, t) implies -m <= f(t) and f(t) <= m })
//     implies continuous_at(integral_function(f, a), x0)
// }
