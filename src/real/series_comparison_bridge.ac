from list import map, partial, sum
from nat import Nat
from real.abs_conv import absolutely_converges, absolutely_converges_comparison,
    absolutely_converges_imp_converges, abs_conv_tail_imp_abs_conv, abs_fn,
    abs_fn_tail_comm
from real.real_base import Real
from real.real_series import comparison_test, converges, is_lower_bound,
    partial_seq_lte, partial_tail_conv_imp_partial_conv, seq_lte, tail,
    tail_partial_converges
from real.series_finite_sum_bridge import real_partial_interval_sum,
    real_partial_tail_eq_interval_sum

numerals Nat

/// If a series converges from some tail onward, then the original series
/// converges.
theorem series_converges_of_tail(f: Nat -> Real, n: Nat) {
    converges(partial(tail(f, n))) implies converges(partial(f))
} by {
    partial_tail_conv_imp_partial_conv(f, n)
}

/// If a series converges, then every tail series converges.
theorem tail_series_converges_of_series(f: Nat -> Real, n: Nat) {
    converges(partial(f)) implies converges(partial(tail(f, n)))
} by {
    if converges(partial(f)) {
        tail_partial_converges(f, n)
        converges(partial(tail(f, n)))
    }
}

/// Series convergence is unchanged by deleting finitely many initial terms.
theorem series_converges_iff_tail(f: Nat -> Real, n: Nat) {
    converges(partial(f)) = converges(partial(tail(f, n)))
} by {
    if converges(partial(f)) {
        tail_series_converges_of_series(f, n)
        converges(partial(tail(f, n)))
    }
    if converges(partial(tail(f, n))) {
        series_converges_of_tail(f, n)
        converges(partial(f))
    }
    converges(partial(f)) = converges(partial(tail(f, n)))
}

/// Absolute convergence from a tail implies absolute convergence of the whole
/// series.
theorem absolutely_converges_of_tail(f: Nat -> Real, n: Nat) {
    absolutely_converges(tail(f, n)) implies absolutely_converges(f)
} by {
    abs_conv_tail_imp_abs_conv(f, n)
}

/// Absolute convergence of a series implies absolute convergence of every tail.
theorem tail_absolutely_converges_of_abs_converges(f: Nat -> Real, n: Nat) {
    absolutely_converges(f) implies absolutely_converges(tail(f, n))
} by {
    if absolutely_converges(f) {
        converges(partial(abs_fn(f)))
        abs_fn_tail_comm(f, n)
        abs_fn(tail(f, n)) = tail(abs_fn(f), n)
        tail_series_converges_of_series(abs_fn(f), n)
        converges(partial(tail(abs_fn(f), n)))
        converges(partial(abs_fn(tail(f, n))))
        absolutely_converges(tail(f, n))
    }
}

/// Absolute convergence is unchanged by deleting finitely many initial terms.
theorem absolutely_converges_iff_tail(f: Nat -> Real, n: Nat) {
    absolutely_converges(f) = absolutely_converges(tail(f, n))
} by {
    if absolutely_converges(f) {
        tail_absolutely_converges_of_abs_converges(f, n)
        absolutely_converges(tail(f, n))
    }
    if absolutely_converges(tail(f, n)) {
        absolutely_converges_of_tail(f, n)
        absolutely_converges(f)
    }
    absolutely_converges(f) = absolutely_converges(tail(f, n))
}

/// Pointwise comparison of two sequences passes to their partial sums.
theorem partial_sums_lte_of_seq_lte(f: Nat -> Real, g: Nat -> Real) {
    seq_lte(f, g) implies seq_lte(partial(f), partial(g))
} by {
    partial_seq_lte(f, g)
}

/// Pointwise comparison of tails passes to their partial sums.
theorem tail_partial_sums_lte_of_tail_seq_lte(f: Nat -> Real, g: Nat -> Real, n: Nat) {
    seq_lte(tail(f, n), tail(g, n)) implies seq_lte(partial(tail(f, n)), partial(tail(g, n)))
} by {
    partial_seq_lte(tail(f, n), tail(g, n))
}

/// A tail comparison can be stated directly with shifted indices.
theorem tail_seq_lte_of_eventual_index_lte(f: Nat -> Real, g: Nat -> Real, n: Nat) {
    (forall(k: Nat) { n <= k implies f(k) <= g(k) }) implies seq_lte(tail(f, n), tail(g, n))
} by {
    if forall(k: Nat) { n <= k implies f(k) <= g(k) } {
        forall(i: Nat) {
            n <= n + i
            f(n + i) <= g(n + i)
            tail(f, n)(i) = f(n + i)
            tail(g, n)(i) = g(n + i)
            tail(f, n)(i) <= tail(g, n)(i)
        }
        seq_lte(tail(f, n), tail(g, n))
    }
}

/// A nonnegative tail can be stated directly with shifted indices.
theorem tail_lower_bound_of_eventual_nonneg(f: Nat -> Real, n: Nat) {
    (forall(k: Nat) { n <= k implies Real.0 <= f(k) }) implies is_lower_bound(tail(f, n), Real.0)
} by {
    if forall(k: Nat) { n <= k implies Real.0 <= f(k) } {
        forall(i: Nat) {
            n <= n + i
            Real.0 <= f(n + i)
            tail(f, n)(i) = f(n + i)
            Real.0 <= tail(f, n)(i)
        }
        is_lower_bound(tail(f, n), Real.0)
    }
}

/// If a nonnegative sequence is eventually dominated by a convergent series,
/// then its series converges.
theorem eventually_dominated_nonneg_series_converges(f: Nat -> Real, g: Nat -> Real, n: Nat) {
    (forall(k: Nat) { n <= k implies Real.0 <= f(k) })
    and (forall(k: Nat) { n <= k implies f(k) <= g(k) })
    and converges(partial(g))
    implies converges(partial(f))
} by {
    if (forall(k: Nat) { n <= k implies Real.0 <= f(k) })
        and (forall(k: Nat) { n <= k implies f(k) <= g(k) })
        and converges(partial(g)) {
        tail_lower_bound_of_eventual_nonneg(f, n)
        is_lower_bound(tail(f, n), Real.0)
        tail_seq_lte_of_eventual_index_lte(f, g, n)
        seq_lte(tail(f, n), tail(g, n))
        tail_series_converges_of_series(g, n)
        converges(partial(tail(g, n)))
        comparison_test(tail(f, n), tail(g, n))
        converges(partial(tail(f, n)))
        converges(partial(f))
    }
}

/// If a sequence is eventually absolutely dominated by a nonnegative
/// convergent majorant, then the sequence is absolutely convergent.
theorem eventually_abs_dominated_absolutely_converges(f: Nat -> Real, g: Nat -> Real, n: Nat) {
    (forall(k: Nat) { n <= k implies Real.0 <= g(k) })
    and (forall(k: Nat) { n <= k implies abs_fn(f)(k) <= g(k) })
    and converges(partial(g))
    implies absolutely_converges(f)
} by {
    if (forall(k: Nat) { n <= k implies Real.0 <= g(k) })
        and (forall(k: Nat) { n <= k implies abs_fn(f)(k) <= g(k) })
        and converges(partial(g)) {
        tail_lower_bound_of_eventual_nonneg(g, n)
        is_lower_bound(tail(g, n), Real.0)
        abs_fn_tail_comm(f, n)
        abs_fn(tail(f, n)) = tail(abs_fn(f), n)
        tail_seq_lte_of_eventual_index_lte(abs_fn(f), g, n)
        seq_lte(tail(abs_fn(f), n), tail(g, n))
        seq_lte(abs_fn(tail(f, n)), tail(g, n))
        tail_series_converges_of_series(g, n)
        converges(partial(tail(g, n)))
        absolutely_converges_comparison(tail(f, n), tail(g, n))
        absolutely_converges(tail(f, n))
        absolutely_converges(f)
    }
}

/// The finite interval sum of a tail is dominated when tail partial sums are
/// dominated.
theorem tail_interval_sum_lte_of_tail_partial_lte(f: Nat -> Real, g: Nat -> Real, n: Nat, k: Nat) {
    seq_lte(partial(tail(f, n)), partial(tail(g, n))) implies
        partial(tail(f, n), k) <= partial(tail(g, n), k)
} by {
    if seq_lte(partial(tail(f, n)), partial(tail(g, n))) {
        seq_lte(partial(tail(f, n)), partial(tail(g, n))) = forall(i: Nat) {
            partial(tail(f, n), i) <= partial(tail(g, n), i)
        }
        partial(tail(f, n), k) <= partial(tail(g, n), k)
    }
}

/// Eventually dominated nonnegative sequences have dominated finite tail
/// interval sums.
theorem eventually_dominated_tail_interval_sum_lte(f: Nat -> Real, g: Nat -> Real, n: Nat, k: Nat) {
    (forall(i: Nat) { n <= i implies f(i) <= g(i) }) implies
        partial(tail(f, n), k) <= partial(tail(g, n), k)
} by {
    if forall(i: Nat) { n <= i implies f(i) <= g(i) } {
        tail_seq_lte_of_eventual_index_lte(f, g, n)
        seq_lte(tail(f, n), tail(g, n))
        partial_sums_lte_of_seq_lte(tail(f, n), tail(g, n))
        seq_lte(partial(tail(f, n)), partial(tail(g, n)))
        tail_interval_sum_lte_of_tail_partial_lte(f, g, n, k)
        partial(tail(f, n), k) <= partial(tail(g, n), k)
    }
}

/// Rewrites finite tail partial-sum comparison as comparison of explicit
/// interval sums.
theorem eventually_dominated_interval_sum_lte(f: Nat -> Real, g: Nat -> Real, n: Nat, k: Nat) {
    (forall(i: Nat) { n <= i implies f(i) <= g(i) }) implies
        sum(map(n.until(n + k), f)) <= sum(map(n.until(n + k), g))
} by {
    if forall(i: Nat) { n <= i implies f(i) <= g(i) } {
        eventually_dominated_tail_interval_sum_lte(f, g, n, k)
        partial(tail(f, n), k) <= partial(tail(g, n), k)
        real_partial_tail_eq_interval_sum(f, n, k)
        partial(tail(f, n), k) = sum(map(n.until(n + k), f))
        real_partial_tail_eq_interval_sum(g, n, k)
        partial(tail(g, n), k) = sum(map(n.until(n + k), g))
        sum(map(n.until(n + k), f)) <= sum(map(n.until(n + k), g))
    }
}

/// Absolute convergence from eventual domination implies ordinary convergence
/// of the dominated series.
theorem eventually_abs_dominated_series_converges(f: Nat -> Real, g: Nat -> Real, n: Nat) {
    (forall(k: Nat) { n <= k implies Real.0 <= g(k) })
    and (forall(k: Nat) { n <= k implies abs_fn(f)(k) <= g(k) })
    and converges(partial(g))
    implies converges(partial(f))
} by {
    if (forall(k: Nat) { n <= k implies Real.0 <= g(k) })
        and (forall(k: Nat) { n <= k implies abs_fn(f)(k) <= g(k) })
        and converges(partial(g)) {
        eventually_abs_dominated_absolutely_converges(f, g, n)
        absolutely_converges(f)
        converges(partial(f))
    }
}
