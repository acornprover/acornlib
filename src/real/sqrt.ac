from order import lte_antisymm, not_lt_imp_gte, not_lte_imp_gt
from real.log import Real, exp_add, exp_log_or_zero, log_mul, log_some_of_pos_exists, rpow_neg_base, rpow_pos, rpow_zero_base_pos
from real.real_base import one_half_plus_one_half, one_half_positive

numerals Real

/// The nonnegative square root, defined on nonnegative reals.
attributes Real {
    /// The nonnegative square root, defined on nonnegative reals.
    define sqrt(self) -> Option[Real] {
        self.rpow(Real.one_half)
    }
}

/// The square root of zero is zero.
theorem sqrt_zero {
    Real.0.sqrt = Option.some(Real.0)
} by {
    one_half_positive
    Real.one_half > Real.0
    rpow_zero_base_pos(Real.one_half)
    (Real.0).rpow(Real.one_half) = Option.some(Real.0)
    Real.0.sqrt = Option.some(Real.0)
}

/// The square root of a positive real is positive.
theorem sqrt_pos(x: Real) {
    x > Real.0 implies exists(y: Real) {
        x.sqrt = Option.some(y) and y > Real.0
    }
} by {
    rpow_pos(x, Real.one_half)
    exists(y: Real) {
        x.rpow(Real.one_half) = Option.some(y) and y > Real.0
    }
    let y: Real satisfy {
        x.rpow(Real.one_half) = Option.some(y) and y > Real.0
    }
    x.sqrt = Option.some(y)
    exists(witness: Real) {
        witness = y and x.sqrt = Option.some(witness) and witness > Real.0
    }
}

/// The square of the square root of a nonnegative real is the original number.
theorem sqrt_mul_self(x: Real) {
    x >= Real.0 implies exists(y: Real) {
        x.sqrt = Option.some(y) and y * y = x
    }
} by {
    if x > Real.0 {
        log_some_of_pos_exists(x)
        let lx: Real satisfy {
            x.log = Option.some(lx)
        }
        let y = (Real.one_half * lx).exp
        option_get_or_else_some[Real](lx, Real.0)
        option_get_or_else(Option.some(lx), Real.0) = lx
        x.log.get_or_else(Real.0) = lx
        x.sqrt = x.rpow(Real.one_half)
        x.rpow(Real.one_half) = Option.some((Real.one_half * x.log.get_or_else(Real.0)).exp)
        x.rpow(Real.one_half) = Option.some((Real.one_half * lx).exp)
        x.sqrt = Option.some((Real.one_half * lx).exp)
        x.sqrt = Option.some(y)
        exp_add(Real.one_half * lx, Real.one_half * lx)
        (Real.one_half * lx + Real.one_half * lx).exp = y * y
        Real.one_half * lx + Real.one_half * lx =
            (Real.one_half + Real.one_half) * lx
        one_half_plus_one_half
        Real.one_half + Real.one_half = Real.1
        (Real.one_half + Real.one_half) * lx = Real.1 * lx
        Real.1 * lx = lx
        Real.one_half * lx + Real.one_half * lx = lx
        y * y = lx.exp
        exp_log_or_zero(x, lx)
        lx.exp = x
        lx.exp = x
        y * y = x
        exists(witness: Real) {
            witness = y and x.sqrt = Option.some(witness) and witness * witness = x
        }
        exists(witness: Real) {
            x.sqrt = Option.some(witness) and witness * witness = x
        }
    } else {
        if not x <= Real.0 {
            not_lte_imp_gt(x, Real.0)
            x > Real.0
            false
        }
        x <= Real.0
        Real.0 <= x
        lte_antisymm(Real.0, x)
        Real.0 = x
        x = Real.0
        sqrt_zero
        x.sqrt = Option.some(Real.0)
        Real.0 * Real.0 = Real.0
        exists(witness: Real) {
            witness = Real.0 and x.sqrt = Option.some(witness) and witness * witness = x
        }
        exists(witness: Real) {
            x.sqrt = Option.some(witness) and witness * witness = x
        }
    }
}

/// The square root of the square of a nonnegative real is the original number.
theorem sqrt_square_of_nonneg(x: Real) {
    x >= Real.0 implies (x * x).sqrt = Option.some(x)
} by {
    if x > Real.0 {
        x.is_positive
        (x * x).is_positive
        x * x > Real.0
        log_some_of_pos_exists(x)
        let lx: Real satisfy {
            x.log = Option.some(lx)
        }
        (x * x).sqrt = Option.some((Real.one_half * (x * x).log.get_or_else(Real.0)).exp)
        log_mul(x, x, lx, lx)
        (x * x).log = Option.some(lx + lx)
        option_get_or_else_some[Real](lx + lx, Real.0)
        option_get_or_else(Option.some(lx + lx), Real.0) = lx + lx
        (x * x).log.get_or_else(Real.0) = lx + lx
        Real.one_half * (x * x).log.get_or_else(Real.0) = Real.one_half * (lx + lx)
        Real.one_half * (lx + lx) = Real.one_half * lx + Real.one_half * lx
        Real.one_half * lx + Real.one_half * lx =
            (Real.one_half + Real.one_half) * lx
        one_half_plus_one_half
        Real.one_half + Real.one_half = Real.1
        (Real.one_half + Real.one_half) * lx = Real.1 * lx
        Real.1 * lx = lx
        Real.one_half * (x * x).log.get_or_else(Real.0) = lx
        (Real.one_half * (x * x).log.get_or_else(Real.0)).exp = lx.exp
        exp_log_or_zero(x, lx)
        lx.exp = x
        lx.exp = x
        (x * x).sqrt = Option.some(x)
    } else {
        if not x <= Real.0 {
            not_lte_imp_gt(x, Real.0)
            x > Real.0
            false
        }
        x <= Real.0
        Real.0 <= x
        lte_antisymm(Real.0, x)
        Real.0 = x
        x = Real.0
        x * x = Real.0
        sqrt_zero
        (x * x).sqrt = Option.some(Real.0)
        (x * x).sqrt = Option.some(x)
    }
}

/// A nonnegative square root is unique.
theorem sqrt_unique_nonneg(x: Real, y: Real) {
    x >= Real.0 and y >= Real.0 and y * y = x implies x.sqrt = Option.some(y)
} by {
    sqrt_square_of_nonneg(y)
    (y * y).sqrt = Option.some(y)
    x.sqrt = (y * y).sqrt
    x.sqrt = Option.some(y)
}
