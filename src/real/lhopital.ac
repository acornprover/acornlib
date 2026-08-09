/// L'Hôpital's rule for the 0/0 indeterminate form at a point.
///
/// This file states and proves L'Hôpital's rule in the sequential limit form
/// used throughout this library: a limit at a point `a` is expressed by saying
/// that every sequence converging to `a` (with terms distinct from `a`) is
/// carried by the function to a sequence converging to the limit value.  The
/// main results are:
///
///   * Cauchy's mean value theorem (`cauchy_mean_value_theorem`), proved from
///     Rolle's theorem by the standard linear-combination argument;
///   * the pointwise 0/0 identity (`lhopital_ratio_cauchy_mvt`): for f(a) = g(a) = 0,
///     the ratio f(x)/g(x) equals the ratio df(c)/dg(c) at some c between a and x;
///   * the continuous-derivatives case of the rule (`lhopital_continuous_derivatives`):
///     if f and g vanish at a, g stays nonzero away from a, and f' and g' are
///     continuous at a with g'(a) != 0, then lim_{x -> a} f(x)/g(x) = f'(a)/g'(a).
///
/// The general statement of the rule (with the limit of df/dg given abstractly)
/// is stated in a comment at the end of the file but not proved.

from data.basic.function_algebra import pointwise_add, pointwise_mul
from data.basic.functions import compose, function_extensionality,
    function_eq_transport_predicate_rev
from nat import Nat
from order_set import closed_interval_set
from real.calculus_api import is_derivative_fn, is_derivative_fn_at,
    derivative_fn_linear_combination
from real.continuity_base import Real, continuous, continuous_at
from real.continuity_composition import continuous_imp_continuous_at,
    constant_function_is_continuous_at
from real.continuity_pointwise import continuous_at_pointwise_add
from real.continuity_pointwise_mul import continuous_at_pointwise_mul
from real.continuity_sequences import continuous_at_preserves_sequence_limit
from real.derivative_basic import has_derivative_at, difference_quotient,
    has_derivative_at_unique, sub_ne_zero_of_ne, forall_elim
from real.derivative_continuity import derivative_continuous_at
from real.derivative_quotient import reciprocal_real, reciprocal_real_apply,
    derivative_reciprocal_real
from real.mean_value import rolle_theorem, continuous_on_closed, differentiable_on_open,
    is_derivative_on_open, is_derivative_on_open_imp_differentiable_on_open
from real.prod_seq import prod_seq, limit_prod_seq
from real.real_base import add_comm, add_assoc, add_zero_left, add_zero_right, neg_zero,
    neg_neg, add_neg_eq_zero, neg_distrib
from real.real_field import div_cancel_common, div_div
from real.real_ring import real_mul_comm, mul_neg_left
from real.real_seq import converges_to, tail_bound, tail_bound_implies_is_close,
    converges, converges_imp_converges_to, converges_to_imp_converges,
    converges_to_unique, sub_zero_imp_eq, limit
from real.asymptotic_bounds import tail_bounds_imp_converges_to
from algebra.field.field import inverse_not_zero, mul_not_zero

numerals Real

/// The pointwise quotient of two real-valued functions.
define quotient_fn(f: Real -> Real, g: Real -> Real, x: Real) -> Real {
    f(x) / g(x)
}

/// True if f has limit `l` at the punctured point a, expressed sequentially:
/// every sequence converging to a with terms distinct from a is mapped to a
/// sequence converging to `l`.
define limit_at(f: Real -> Real, a: Real, l: Real) -> Bool {
    forall(q: Nat -> Real) {
        (forall(n: Nat) { q(n) != a }) and converges_to(q, a)
        implies converges_to(compose(f, q), l)
    }
}

// =============================================================================
// Cauchy's mean value theorem
// =============================================================================

/// The linear combination (f(b) - f(a)) * g - (g(b) - g(a)) * f, whose Rolle
/// point has a zero derivative and yields Cauchy's mean value theorem.
define cauchy_combination(f: Real -> Real, g: Real -> Real, a: Real, b: Real, x: Real) -> Real {
    (f(b) - f(a)) * g(x) - (g(b) - g(a)) * f(x)
}

/// The pointwise derivative of the Cauchy combination.
define cauchy_combination_derivative(f: Real -> Real, g: Real -> Real, df: Real -> Real,
    dg: Real -> Real, a: Real, b: Real, x: Real) -> Real {
    (f(b) - f(a)) * dg(x) - (g(b) - g(a)) * df(x)
}

/// The Cauchy combination is the pointwise sum of two constant multiples.
theorem cauchy_combination_eq_pointwise(f: Real -> Real, g: Real -> Real, a: Real, b: Real) {
    cauchy_combination(f, g, a, b) = pointwise_add(
        pointwise_mul(constant[Real, Real](f(b) - f(a)), g),
        pointwise_mul(constant[Real, Real](-(g(b) - g(a))), f))
} by {
    forall(x: Real) {
        cauchy_combination(f, g, a, b, x) = (f(b) - f(a)) * g(x) - (g(b) - g(a)) * f(x)
        (f(b) - f(a)) * g(x) - (g(b) - g(a)) * f(x) =
            (f(b) - f(a)) * g(x) + -((g(b) - g(a)) * f(x))
        mul_neg_left(g(b) - g(a), f(x))
        -(g(b) - g(a)) * f(x) = -((g(b) - g(a)) * f(x))
        (f(b) - f(a)) * g(x) - (g(b) - g(a)) * f(x) =
            (f(b) - f(a)) * g(x) + -(g(b) - g(a)) * f(x)
        pointwise_mul(constant[Real, Real](f(b) - f(a)), g, x) =
            constant[Real, Real](f(b) - f(a), x) * g(x)
        constant[Real, Real](f(b) - f(a), x) = f(b) - f(a)
        pointwise_mul(constant[Real, Real](f(b) - f(a)), g, x) = (f(b) - f(a)) * g(x)
        pointwise_mul(constant[Real, Real](-(g(b) - g(a))), f, x) =
            constant[Real, Real](-(g(b) - g(a)), x) * f(x)
        constant[Real, Real](-(g(b) - g(a)), x) = -(g(b) - g(a))
        pointwise_mul(constant[Real, Real](-(g(b) - g(a))), f, x) = -(g(b) - g(a)) * f(x)
        pointwise_add(
            pointwise_mul(constant[Real, Real](f(b) - f(a)), g),
            pointwise_mul(constant[Real, Real](-(g(b) - g(a))), f), x) =
            pointwise_mul(constant[Real, Real](f(b) - f(a)), g, x) +
            pointwise_mul(constant[Real, Real](-(g(b) - g(a))), f, x)
        pointwise_add(
            pointwise_mul(constant[Real, Real](f(b) - f(a)), g),
            pointwise_mul(constant[Real, Real](-(g(b) - g(a))), f), x) =
            (f(b) - f(a)) * g(x) + -(g(b) - g(a)) * f(x)
        cauchy_combination(f, g, a, b, x) =
            pointwise_add(
                pointwise_mul(constant[Real, Real](f(b) - f(a)), g),
                pointwise_mul(constant[Real, Real](-(g(b) - g(a))), f), x)
    }
    function_extensionality(cauchy_combination(f, g, a, b),
        pointwise_add(
            pointwise_mul(constant[Real, Real](f(b) - f(a)), g),
            pointwise_mul(constant[Real, Real](-(g(b) - g(a))), f)))
    cauchy_combination(f, g, a, b) = pointwise_add(
        pointwise_mul(constant[Real, Real](f(b) - f(a)), g),
        pointwise_mul(constant[Real, Real](-(g(b) - g(a))), f))
}

/// The Cauchy combination is continuous on the closed interval when f and g are.
theorem cauchy_combination_continuous_on_closed(f: Real -> Real, g: Real -> Real,
    a: Real, b: Real) {
    continuous(f) and continuous(g)
    implies continuous_on_closed(cauchy_combination(f, g, a, b), a, b)
} by {
    if continuous(f) and continuous(g) {
        forall(x: Real) {
            if closed_interval_set(a, b).contains(x) {
                continuous_imp_continuous_at(f, x)
                continuous_at(f, x)
                continuous_imp_continuous_at(g, x)
                continuous_at(g, x)
                constant_function_is_continuous_at(f(b) - f(a), x)
                continuous_at(constant[Real, Real](f(b) - f(a)), x)
                continuous_at_pointwise_mul(constant[Real, Real](f(b) - f(a)), g, x)
                continuous_at(pointwise_mul(constant[Real, Real](f(b) - f(a)), g), x)
                constant_function_is_continuous_at(-(g(b) - g(a)), x)
                continuous_at(constant[Real, Real](-(g(b) - g(a))), x)
                continuous_at_pointwise_mul(constant[Real, Real](-(g(b) - g(a))), f, x)
                continuous_at(pointwise_mul(constant[Real, Real](-(g(b) - g(a))), f), x)
                continuous_at_pointwise_add(
                    pointwise_mul(constant[Real, Real](f(b) - f(a)), g),
                    pointwise_mul(constant[Real, Real](-(g(b) - g(a))), f), x)
                continuous_at(pointwise_add(
                    pointwise_mul(constant[Real, Real](f(b) - f(a)), g),
                    pointwise_mul(constant[Real, Real](-(g(b) - g(a))), f)), x)
                cauchy_combination_eq_pointwise(f, g, a, b)
                cauchy_combination(f, g, a, b) = pointwise_add(
                    pointwise_mul(constant[Real, Real](f(b) - f(a)), g),
                    pointwise_mul(constant[Real, Real](-(g(b) - g(a))), f))
                function_eq_transport_predicate_rev(
                    function(h: Real -> Real) { continuous_at(h, x) },
                    cauchy_combination(f, g, a, b),
                    pointwise_add(
                        pointwise_mul(constant[Real, Real](f(b) - f(a)), g),
                        pointwise_mul(constant[Real, Real](-(g(b) - g(a))), f)))
                continuous_at(cauchy_combination(f, g, a, b), x)
            }
        }
        continuous_on_closed(cauchy_combination(f, g, a, b), a, b) = forall(x: Real) {
            closed_interval_set(a, b).contains(x) implies
                continuous_at(cauchy_combination(f, g, a, b), x)
        }
        continuous_on_closed(cauchy_combination(f, g, a, b), a, b)
    }
}

/// The Cauchy combination is differentiable on the open interval with the
/// expected pointwise derivative.
theorem cauchy_combination_is_derivative_on_open(f: Real -> Real, g: Real -> Real,
    df: Real -> Real, dg: Real -> Real, a: Real, b: Real) {
    is_derivative_fn(f, df) and is_derivative_fn(g, dg)
    implies is_derivative_on_open(cauchy_combination(f, g, a, b),
        cauchy_combination_derivative(f, g, df, dg, a, b), a, b)
} by {
    if is_derivative_fn(f, df) and is_derivative_fn(g, dg) {
        derivative_fn_linear_combination(f(b) - f(a), -(g(b) - g(a)), g, f, dg, df)
        is_derivative_fn(
            pointwise_add(
                pointwise_mul(constant[Real, Real](f(b) - f(a)), g),
                pointwise_mul(constant[Real, Real](-(g(b) - g(a))), f)),
            pointwise_add(
                pointwise_mul(constant[Real, Real](f(b) - f(a)), dg),
                pointwise_mul(constant[Real, Real](-(g(b) - g(a))), df)))
        forall(x: Real) {
            if a < x and x < b {
                is_derivative_fn_at(
                    pointwise_add(
                        pointwise_mul(constant[Real, Real](f(b) - f(a)), g),
                        pointwise_mul(constant[Real, Real](-(g(b) - g(a))), f)),
                    pointwise_add(
                        pointwise_mul(constant[Real, Real](f(b) - f(a)), dg),
                        pointwise_mul(constant[Real, Real](-(g(b) - g(a))), df)), x)
                has_derivative_at(
                    pointwise_add(
                        pointwise_mul(constant[Real, Real](f(b) - f(a)), g),
                        pointwise_mul(constant[Real, Real](-(g(b) - g(a))), f)),
                    x,
                    pointwise_add(
                        pointwise_mul(constant[Real, Real](f(b) - f(a)), dg),
                        pointwise_mul(constant[Real, Real](-(g(b) - g(a))), df), x))
                pointwise_mul(constant[Real, Real](f(b) - f(a)), dg, x) =
                    constant[Real, Real](f(b) - f(a), x) * dg(x)
                constant[Real, Real](f(b) - f(a), x) = f(b) - f(a)
                pointwise_mul(constant[Real, Real](f(b) - f(a)), dg, x) =
                    (f(b) - f(a)) * dg(x)
                pointwise_mul(constant[Real, Real](-(g(b) - g(a))), df, x) =
                    constant[Real, Real](-(g(b) - g(a)), x) * df(x)
                constant[Real, Real](-(g(b) - g(a)), x) = -(g(b) - g(a))
                pointwise_mul(constant[Real, Real](-(g(b) - g(a))), df, x) =
                    -(g(b) - g(a)) * df(x)
                pointwise_add(
                    pointwise_mul(constant[Real, Real](f(b) - f(a)), dg),
                    pointwise_mul(constant[Real, Real](-(g(b) - g(a))), df), x) =
                    pointwise_mul(constant[Real, Real](f(b) - f(a)), dg, x) +
                    pointwise_mul(constant[Real, Real](-(g(b) - g(a))), df, x)
                pointwise_add(
                    pointwise_mul(constant[Real, Real](f(b) - f(a)), dg),
                    pointwise_mul(constant[Real, Real](-(g(b) - g(a))), df), x) =
                    (f(b) - f(a)) * dg(x) + -(g(b) - g(a)) * df(x)
                cauchy_combination_derivative(f, g, df, dg, a, b, x) =
                    (f(b) - f(a)) * dg(x) - (g(b) - g(a)) * df(x)
                (f(b) - f(a)) * dg(x) - (g(b) - g(a)) * df(x) =
                    (f(b) - f(a)) * dg(x) + -((g(b) - g(a)) * df(x))
                mul_neg_left(g(b) - g(a), df(x))
                -(g(b) - g(a)) * df(x) = -((g(b) - g(a)) * df(x))
                (f(b) - f(a)) * dg(x) - (g(b) - g(a)) * df(x) =
                    (f(b) - f(a)) * dg(x) + -(g(b) - g(a)) * df(x)
                cauchy_combination_derivative(f, g, df, dg, a, b, x) =
                    pointwise_add(
                        pointwise_mul(constant[Real, Real](f(b) - f(a)), dg),
                        pointwise_mul(constant[Real, Real](-(g(b) - g(a))), df), x)
                has_derivative_at(
                    pointwise_add(
                        pointwise_mul(constant[Real, Real](f(b) - f(a)), g),
                        pointwise_mul(constant[Real, Real](-(g(b) - g(a))), f)),
                    x, cauchy_combination_derivative(f, g, df, dg, a, b, x))
                cauchy_combination_eq_pointwise(f, g, a, b)
                cauchy_combination(f, g, a, b) = pointwise_add(
                    pointwise_mul(constant[Real, Real](f(b) - f(a)), g),
                    pointwise_mul(constant[Real, Real](-(g(b) - g(a))), f))
                function_eq_transport_predicate_rev(
                    function(h: Real -> Real) {
                        has_derivative_at(h, x, cauchy_combination_derivative(f, g, df, dg, a, b, x))
                    },
                    cauchy_combination(f, g, a, b),
                    pointwise_add(
                        pointwise_mul(constant[Real, Real](f(b) - f(a)), g),
                        pointwise_mul(constant[Real, Real](-(g(b) - g(a))), f)))
                has_derivative_at(cauchy_combination(f, g, a, b), x,
                    cauchy_combination_derivative(f, g, df, dg, a, b, x))
            }
        }
        is_derivative_on_open(cauchy_combination(f, g, a, b),
            cauchy_combination_derivative(f, g, df, dg, a, b), a, b) = forall(x: Real) {
            a < x and x < b implies has_derivative_at(cauchy_combination(f, g, a, b), x,
                cauchy_combination_derivative(f, g, df, dg, a, b, x))
        }
        is_derivative_on_open(cauchy_combination(f, g, a, b),
            cauchy_combination_derivative(f, g, df, dg, a, b), a, b)
    }
}

/// Negating a difference negates the first term and keeps the second.
theorem neg_sub_distrib(x: Real, y: Real) {
    -(x - y) = -x + y
} by {
    x - y = x + -y
    neg_distrib(x, -y)
    -(x + -y) = -x + -(-y)
    neg_neg(y)
    -(-y) = y
    -x + -(-y) = -x + y
    -(x - y) = -x + y
}

/// Subtracting a difference distributes the negation over both terms.
theorem sub_sub_distrib(x: Real, y: Real, z: Real) {
    x - (y - z) = x - y + z
} by {
    x - (y - z) = x + -(y - z)
    neg_sub_distrib(y, z)
    -(y - z) = -y + z
    x + -(y - z) = x + (-y + z)
    add_assoc(x, -y, z)
    (x + -y) + z = x + (-y + z)
    x + (-y + z) = (x + -y) + z
    x - y = x + -y
    (x + -y) + z = x - y + z
    x - (y - z) = x - y + z
}

/// A real number minus itself is zero.
theorem sub_self_zero(x: Real) {
    x - x = Real.0
} by {
    x - x = x + -x
    add_neg_eq_zero(x)
    x + -x = Real.0
    x - x = Real.0
}

/// The Cauchy combination takes equal values at the endpoints.
theorem cauchy_combination_endpoints_equal(f: Real -> Real, g: Real -> Real,
    a: Real, b: Real) {
    cauchy_combination(f, g, a, b, a) = cauchy_combination(f, g, a, b, b)
} by {
    // Both endpoint values expand to f(b) * g(a) - f(a) * g(b).
    cauchy_combination(f, g, a, b, a) = (f(b) - f(a)) * g(a) - (g(b) - g(a)) * f(a)
    (f(b) - f(a)) * g(a) = f(b) * g(a) - f(a) * g(a)
    (g(b) - g(a)) * f(a) = g(b) * f(a) - g(a) * f(a)
    (f(b) - f(a)) * g(a) - (g(b) - g(a)) * f(a) =
        (f(b) * g(a) - f(a) * g(a)) - (g(b) * f(a) - g(a) * f(a))
    sub_sub_distrib(f(b) * g(a) - f(a) * g(a), g(b) * f(a), g(a) * f(a))
    (f(b) * g(a) - f(a) * g(a)) - (g(b) * f(a) - g(a) * f(a)) =
        f(b) * g(a) - f(a) * g(a) - g(b) * f(a) + g(a) * f(a)
    real_mul_comm(g(a), f(a))
    g(a) * f(a) = f(a) * g(a)
    f(b) * g(a) - f(a) * g(a) - g(b) * f(a) + g(a) * f(a) =
        f(b) * g(a) - f(a) * g(a) - g(b) * f(a) + f(a) * g(a)
    f(b) * g(a) - f(a) * g(a) - g(b) * f(a) + f(a) * g(a) =
        f(b) * g(a) - g(b) * f(a) - f(a) * g(a) + f(a) * g(a)
    f(b) * g(a) - g(b) * f(a) - f(a) * g(a) + f(a) * g(a) =
        f(b) * g(a) - g(b) * f(a) + (-(f(a) * g(a)) + f(a) * g(a))
    add_neg_eq_zero(f(a) * g(a))
    f(a) * g(a) + -(f(a) * g(a)) = Real.0
    add_comm(f(a) * g(a), -(f(a) * g(a)))
    f(a) * g(a) + -(f(a) * g(a)) = -(f(a) * g(a)) + f(a) * g(a)
    -(f(a) * g(a)) + f(a) * g(a) = Real.0
    f(b) * g(a) - g(b) * f(a) + (-(f(a) * g(a)) + f(a) * g(a)) =
        f(b) * g(a) - g(b) * f(a) + Real.0
    add_zero_right(f(b) * g(a) - g(b) * f(a))
    f(b) * g(a) - g(b) * f(a) + Real.0 = f(b) * g(a) - g(b) * f(a)
    real_mul_comm(g(b), f(a))
    g(b) * f(a) = f(a) * g(b)
    f(b) * g(a) - g(b) * f(a) = f(b) * g(a) - f(a) * g(b)
    cauchy_combination(f, g, a, b, a) = f(b) * g(a) - f(a) * g(b)

    cauchy_combination(f, g, a, b, b) = (f(b) - f(a)) * g(b) - (g(b) - g(a)) * f(b)
    (f(b) - f(a)) * g(b) = f(b) * g(b) - f(a) * g(b)
    (g(b) - g(a)) * f(b) = g(b) * f(b) - g(a) * f(b)
    (f(b) - f(a)) * g(b) - (g(b) - g(a)) * f(b) =
        (f(b) * g(b) - f(a) * g(b)) - (g(b) * f(b) - g(a) * f(b))
    sub_sub_distrib(f(b) * g(b) - f(a) * g(b), g(b) * f(b), g(a) * f(b))
    (f(b) * g(b) - f(a) * g(b)) - (g(b) * f(b) - g(a) * f(b)) =
        f(b) * g(b) - f(a) * g(b) - g(b) * f(b) + g(a) * f(b)
    real_mul_comm(g(b), f(b))
    g(b) * f(b) = f(b) * g(b)
    f(b) * g(b) - f(a) * g(b) - g(b) * f(b) + g(a) * f(b) =
        f(b) * g(b) - f(a) * g(b) - f(b) * g(b) + g(a) * f(b)
    f(b) * g(b) - f(a) * g(b) - f(b) * g(b) + g(a) * f(b) =
        f(b) * g(b) - f(b) * g(b) - f(a) * g(b) + g(a) * f(b)
    sub_self_zero(f(b) * g(b))
    f(b) * g(b) - f(b) * g(b) = Real.0
    f(b) * g(b) - f(b) * g(b) - f(a) * g(b) + g(a) * f(b) =
        Real.0 - f(a) * g(b) + g(a) * f(b)
    add_zero_left(g(a) * f(b) - f(a) * g(b))
    Real.0 + (g(a) * f(b) - f(a) * g(b)) = g(a) * f(b) - f(a) * g(b)
    Real.0 - f(a) * g(b) + g(a) * f(b) = g(a) * f(b) - f(a) * g(b)
    real_mul_comm(g(a), f(b))
    g(a) * f(b) = f(b) * g(a)
    g(a) * f(b) - f(a) * g(b) = f(b) * g(a) - f(a) * g(b)
    cauchy_combination(f, g, a, b, b) = f(b) * g(a) - f(a) * g(b)
    cauchy_combination(f, g, a, b, a) = cauchy_combination(f, g, a, b, b)
}

/// Cauchy's mean value theorem: for two differentiable functions on [a, b],
/// some interior point has (f(b) - f(a)) * g'(c) = (g(b) - g(a)) * f'(c).
theorem cauchy_mean_value_theorem(f: Real -> Real, g: Real -> Real, df: Real -> Real,
    dg: Real -> Real, a: Real, b: Real) {
    continuous(f) and continuous(g) and a < b and is_derivative_fn(f, df) and
    is_derivative_fn(g, dg)
    implies exists(c: Real) {
        a < c and c < b and (f(b) - f(a)) * dg(c) = (g(b) - g(a)) * df(c)
    }
} by {
    if continuous(f) and continuous(g) and a < b and is_derivative_fn(f, df) and
       is_derivative_fn(g, dg) {
        cauchy_combination_continuous_on_closed(f, g, a, b)
        continuous_on_closed(cauchy_combination(f, g, a, b), a, b)
        cauchy_combination_is_derivative_on_open(f, g, df, dg, a, b)
        is_derivative_on_open(cauchy_combination(f, g, a, b),
            cauchy_combination_derivative(f, g, df, dg, a, b), a, b)
        is_derivative_on_open_imp_differentiable_on_open(cauchy_combination(f, g, a, b),
            cauchy_combination_derivative(f, g, df, dg, a, b), a, b)
        differentiable_on_open(cauchy_combination(f, g, a, b), a, b)
        cauchy_combination_endpoints_equal(f, g, a, b)
        cauchy_combination(f, g, a, b, a) = cauchy_combination(f, g, a, b, b)
        rolle_theorem(cauchy_combination(f, g, a, b), a, b)
        let c: Real satisfy {
            a < c and c < b and has_derivative_at(cauchy_combination(f, g, a, b), c, Real.0)
        }
        a < c and c < b
        has_derivative_at(cauchy_combination(f, g, a, b), c, Real.0)
        is_derivative_on_open(cauchy_combination(f, g, a, b),
            cauchy_combination_derivative(f, g, df, dg, a, b), a, b) = forall(x: Real) {
            a < x and x < b implies has_derivative_at(cauchy_combination(f, g, a, b), x,
                cauchy_combination_derivative(f, g, df, dg, a, b, x))
        }
        a < c and c < b implies has_derivative_at(cauchy_combination(f, g, a, b), c,
            cauchy_combination_derivative(f, g, df, dg, a, b, c))
        has_derivative_at(cauchy_combination(f, g, a, b), c,
            cauchy_combination_derivative(f, g, df, dg, a, b, c))
        has_derivative_at_unique(cauchy_combination(f, g, a, b), c, Real.0,
            cauchy_combination_derivative(f, g, df, dg, a, b, c))
        Real.0 = cauchy_combination_derivative(f, g, df, dg, a, b, c)
        cauchy_combination_derivative(f, g, df, dg, a, b, c) = Real.0
        cauchy_combination_derivative(f, g, df, dg, a, b, c) =
            (f(b) - f(a)) * dg(c) - (g(b) - g(a)) * df(c)
        (f(b) - f(a)) * dg(c) - (g(b) - g(a)) * df(c) = Real.0
        sub_zero_imp_eq((f(b) - f(a)) * dg(c), (g(b) - g(a)) * df(c))
        (f(b) - f(a)) * dg(c) = (g(b) - g(a)) * df(c)
        exists(c2: Real) {
            a < c2 and c2 < b and (f(b) - f(a)) * dg(c2) = (g(b) - g(a)) * df(c2)
        }
    }
}

// =============================================================================
// The pointwise 0/0 ratio via Cauchy's mean value theorem
// =============================================================================

/// L'Hôpital's rule, pointwise form: for f(a) = g(a) = 0, the ratio f(x)/g(x)
/// equals the ratio df(c)/dg(c) for some c strictly between a and x, provided
/// the denominators are nonzero.
theorem lhopital_ratio_cauchy_mvt(f: Real -> Real, g: Real -> Real, df: Real -> Real,
    dg: Real -> Real, a: Real, x: Real) {
    continuous(f) and continuous(g) and is_derivative_fn(f, df) and
    is_derivative_fn(g, dg) and f(a) = Real.0 and g(a) = Real.0 and a < x and
    g(x) != Real.0 and (forall(z: Real) { a < z and z < x implies dg(z) != Real.0 })
    implies exists(c: Real) {
        a < c and c < x and quotient_fn(f, g, x) = quotient_fn(df, dg, c)
    }
} by {
    if continuous(f) and continuous(g) and is_derivative_fn(f, df) and
       is_derivative_fn(g, dg) and f(a) = Real.0 and g(a) = Real.0 and a < x and
       g(x) != Real.0 and (forall(z: Real) { a < z and z < x implies dg(z) != Real.0 }) {
        cauchy_mean_value_theorem(f, g, df, dg, a, x)
        let c: Real satisfy {
            a < c and c < x and (f(x) - f(a)) * dg(c) = (g(x) - g(a)) * df(c)
        }
        a < c and c < x
        (f(x) - f(a)) * dg(c) = (g(x) - g(a)) * df(c)
        f(x) - f(a) = f(x) - Real.0
        f(x) - Real.0 = f(x) + -Real.0
        neg_zero
        -Real.0 = Real.0
        f(x) + -Real.0 = f(x) + Real.0
        add_zero_right(f(x))
        f(x) + Real.0 = f(x)
        f(x) - Real.0 = f(x)
        f(x) - f(a) = f(x)
        g(x) - g(a) = g(x) - Real.0
        g(x) - Real.0 = g(x) + -Real.0
        g(x) + -Real.0 = g(x) + Real.0
        g(x) + Real.0 = g(x)
        g(x) - Real.0 = g(x)
        g(x) - g(a) = g(x)
        f(x) * dg(c) = g(x) * df(c)
        forall(z: Real) { a < z and z < x implies dg(z) != Real.0 }
        a < c and c < x implies dg(c) != Real.0
        dg(c) != Real.0
        div_cancel_common(f(x), dg(c), g(x))
        (f(x) * dg(c)) / (g(x) * dg(c)) = f(x) / g(x)
        div_cancel_common(df(c), g(x), dg(c))
        (df(c) * g(x)) / (dg(c) * g(x)) = df(c) / dg(c)
        real_mul_comm(g(x), df(c))
        g(x) * df(c) = df(c) * g(x)
        f(x) * dg(c) = df(c) * g(x)
        (f(x) * dg(c)) / (g(x) * dg(c)) = (df(c) * g(x)) / (g(x) * dg(c))
        real_mul_comm(g(x), dg(c))
        g(x) * dg(c) = dg(c) * g(x)
        (df(c) * g(x)) / (g(x) * dg(c)) = (df(c) * g(x)) / (dg(c) * g(x))
        (f(x) * dg(c)) / (g(x) * dg(c)) = (df(c) * g(x)) / (dg(c) * g(x))
        f(x) / g(x) = df(c) / dg(c)
        quotient_fn(f, g, x) = f(x) / g(x)
        quotient_fn(df, dg, c) = df(c) / dg(c)
        quotient_fn(f, g, x) = quotient_fn(df, dg, c)
        exists(c2: Real) {
            a < c2 and c2 < x and quotient_fn(f, g, x) = quotient_fn(df, dg, c2)
        }
    }
}

// =============================================================================
// Sequential limits of difference quotients and quotients
// =============================================================================

/// A quotient of two nonzero reals is nonzero.
theorem div_not_zero(u: Real, v: Real) {
    u != Real.0 and v != Real.0 implies u / v != Real.0
} by {
    if u != Real.0 and v != Real.0 {
        inverse_not_zero[Real](v)
        v.inverse != Real.0
        mul_not_zero[Real](u, v.inverse)
        u * v.inverse != Real.0
        u / v = u * v.inverse
        u / v != Real.0
    }
}

/// A derivative at a point makes the composed difference quotients admit a
/// convergence tail bound for every tolerance along any sequence converging
/// to the point.
theorem dq_seq_tail_bounds(f: Real -> Real, a: Real, d: Real, q: Nat -> Real) {
    has_derivative_at(f, a, d) and converges_to(q, a) and
    (forall(n: Nat) { q(n) != a })
    implies forall(eps2: Real) {
        eps2.is_positive implies exists(n2: Nat) {
            tail_bound(compose(difference_quotient(f, a), q), d, n2, eps2)
        }
    }
} by {
    has_derivative_at(f, a, d) = forall(eps3: Real) {
        eps3.is_positive implies exists(delta2: Real) {
            delta2.is_positive and forall(x2: Real) {
                x2 != a and x2.is_close(a, delta2)
                implies difference_quotient(f, a, x2).is_close(d, eps3)
            }
        }
    }
    converges_to(q, a) = forall(eps3: Real) {
        eps3.is_positive implies exists(n3: Nat) {
            tail_bound(q, a, n3, eps3)
        }
    }
    if has_derivative_at(f, a, d) and converges_to(q, a) and
       (forall(n: Nat) { q(n) != a }) {
        forall(eps: Real) {
            if eps.is_positive {
                let delta: Real satisfy {
                    delta.is_positive and forall(x2: Real) {
                        x2 != a and x2.is_close(a, delta)
                        implies difference_quotient(f, a, x2).is_close(d, eps)
                    }
                }
                let n: Nat satisfy {
                    tail_bound(q, a, n, delta)
                }
                forall(i: Nat) {
                    if n <= i {
                        tail_bound_implies_is_close(q, a, n, delta, i)
                        q(i).is_close(a, delta)
                        forall(n0: Nat) { q(n0) != a }
                        q(i) != a
                        q(i) != a and q(i).is_close(a, delta)
                        forall_elim[Real](function(x2: Real) {
                            x2 != a and x2.is_close(a, delta) implies
                                difference_quotient(f, a, x2).is_close(d, eps)
                        }, q(i))
                        function(x2: Real) {
                            x2 != a and x2.is_close(a, delta) implies
                                difference_quotient(f, a, x2).is_close(d, eps)
                        }(q(i))
                        difference_quotient(f, a, q(i)).is_close(d, eps)
                        compose(difference_quotient(f, a), q, i) =
                            difference_quotient(f, a, q(i))
                        compose(difference_quotient(f, a), q, i).is_close(d, eps)
                    }
                }
                tail_bound(compose(difference_quotient(f, a), q), d, n, eps)
            }
        }
    }
}

/// A derivative at a point makes the composed difference quotients converge to
/// the derivative value along any sequence converging to the point.
theorem difference_quotient_seq_limit(f: Real -> Real, a: Real, d: Real, q: Nat -> Real) {
    has_derivative_at(f, a, d) and converges_to(q, a) and
    (forall(n: Nat) { q(n) != a })
    implies converges_to(compose(difference_quotient(f, a), q), d)
} by {
    if has_derivative_at(f, a, d) and converges_to(q, a) and
       (forall(n: Nat) { q(n) != a }) {
        dq_seq_tail_bounds(f, a, d, q)
        tail_bounds_imp_converges_to(compose(difference_quotient(f, a), q), d)
    }
}

/// Along a sequence approaching a with g(a) = 0, the denominator difference
/// quotients of g are nonzero.
theorem dq_denominator_seq_nonzero(g: Real -> Real, a: Real, q: Nat -> Real) {
    g(a) = Real.0 and (forall(n: Nat) { q(n) != a }) and
    (forall(x: Real) { x != a implies g(x) != Real.0 })
    implies forall(n: Nat) { compose(difference_quotient(g, a), q, n) != Real.0 }
} by {
    if g(a) = Real.0 and (forall(n: Nat) { q(n) != a }) and
       (forall(x: Real) { x != a implies g(x) != Real.0 }) {
        forall(n: Nat) {
            compose(difference_quotient(g, a), q, n) = difference_quotient(g, a, q(n))
            difference_quotient(g, a, q(n)) = (g(q(n)) - g(a)) / (q(n) - a)
            g(q(n)) - g(a) = g(q(n)) - Real.0
            g(q(n)) - Real.0 = g(q(n)) + -Real.0
            neg_zero
            -Real.0 = Real.0
            g(q(n)) + -Real.0 = g(q(n)) + Real.0
            add_zero_right(g(q(n)))
            g(q(n)) + Real.0 = g(q(n))
            g(q(n)) - Real.0 = g(q(n))
            g(q(n)) - g(a) = g(q(n))
            difference_quotient(g, a, q(n)) = g(q(n)) / (q(n) - a)
            forall(n0: Nat) { q(n0) != a }
            q(n) != a
            forall(x: Real) { x != a implies g(x) != Real.0 }
            q(n) != a implies g(q(n)) != Real.0
            g(q(n)) != Real.0
            sub_ne_zero_of_ne(q(n), a)
            q(n) - a != Real.0
            div_not_zero(g(q(n)), q(n) - a)
            g(q(n)) / (q(n) - a) != Real.0
            difference_quotient(g, a, q(n)) != Real.0
            compose(difference_quotient(g, a), q, n) != Real.0
        }
    }
}

/// Away from the base point, the ratio of difference quotients equals the
/// ratio of the functions when both functions vanish at the base point.
theorem lhopital_pointwise_dq_ratio(f: Real -> Real, g: Real -> Real, a: Real, x: Real) {
    f(a) = Real.0 and g(a) = Real.0 and x != a and g(x) != Real.0
    implies difference_quotient(f, a, x) / difference_quotient(g, a, x) =
        quotient_fn(f, g, x)
} by {
    if f(a) = Real.0 and g(a) = Real.0 and x != a and g(x) != Real.0 {
        difference_quotient(f, a, x) = (f(x) - f(a)) / (x - a)
        f(x) - f(a) = f(x) - Real.0
        f(x) - Real.0 = f(x) + -Real.0
        neg_zero
        -Real.0 = Real.0
        f(x) + -Real.0 = f(x) + Real.0
        add_zero_right(f(x))
        f(x) + Real.0 = f(x)
        f(x) - Real.0 = f(x)
        f(x) - f(a) = f(x)
        difference_quotient(f, a, x) = f(x) / (x - a)
        difference_quotient(g, a, x) = (g(x) - g(a)) / (x - a)
        g(x) - g(a) = g(x) - Real.0
        g(x) - Real.0 = g(x) + -Real.0
        g(x) + -Real.0 = g(x) + Real.0
        g(x) + Real.0 = g(x)
        g(x) - Real.0 = g(x)
        g(x) - g(a) = g(x)
        difference_quotient(g, a, x) = g(x) / (x - a)
        sub_ne_zero_of_ne(x, a)
        x - a != Real.0
        div_div(f(x), x - a, g(x), x - a)
        (f(x) / (x - a)) / (g(x) / (x - a)) = (f(x) * (x - a)) / ((x - a) * g(x))
        real_mul_comm(x - a, g(x))
        (x - a) * g(x) = g(x) * (x - a)
        (f(x) * (x - a)) / ((x - a) * g(x)) =
            (f(x) * (x - a)) / (g(x) * (x - a))
        div_cancel_common(f(x), x - a, g(x))
        (f(x) * (x - a)) / (g(x) * (x - a)) = f(x) / g(x)
        (f(x) / (x - a)) / (g(x) / (x - a)) = f(x) / g(x)
        difference_quotient(f, a, x) / difference_quotient(g, a, x) = f(x) / g(x)
        quotient_fn(f, g, x) = f(x) / g(x)
        difference_quotient(f, a, x) / difference_quotient(g, a, x) = quotient_fn(f, g, x)
    }
}

/// Pointwise equal sequences converge to the same limit.
theorem seq_pointwise_eq_converges_to(u: Nat -> Real, v: Nat -> Real, x: Real) {
    (forall(n: Nat) { u(n) = v(n) }) and converges_to(u, x)
    implies converges_to(v, x)
} by {
    if (forall(n: Nat) { u(n) = v(n) }) and converges_to(u, x) {
        function_extensionality(u, v)
        u = v
        function_eq_transport_predicate_rev(function(w: Nat -> Real) { converges_to(w, x) }, u, v)
        converges_to(v, x)
    }
}

/// The termwise reciprocal of a sequence.
define recip_seq(b: Nat -> Real, n: Nat) -> Real {
    b(n).inverse
}

/// A sequence converging to a nonzero value has termwise reciprocals converging
/// to the reciprocal value.  This is the continuity of the reciprocal function
/// at a nonzero point, expressed sequentially.
theorem converges_to_recip_seq(b: Nat -> Real, b_lim: Real) {
    converges_to(b, b_lim) and b_lim != Real.0
    implies converges_to(recip_seq(b), b_lim.inverse)
} by {
    if converges_to(b, b_lim) and b_lim != Real.0 {
        derivative_reciprocal_real(b_lim)
        has_derivative_at(reciprocal_real, b_lim, -Real.1 / (b_lim * b_lim))
        derivative_continuous_at(reciprocal_real, b_lim, -Real.1 / (b_lim * b_lim))
        continuous_at(reciprocal_real, b_lim)
        continuous_at_preserves_sequence_limit(reciprocal_real, b, b_lim)
        converges_to(compose(reciprocal_real, b), reciprocal_real(b_lim))
        forall(n: Nat) {
            compose(reciprocal_real, b, n) = reciprocal_real(b(n))
            reciprocal_real_apply(b(n))
            reciprocal_real(b(n)) = b(n).inverse
            compose(reciprocal_real, b, n) = b(n).inverse
            recip_seq(b, n) = b(n).inverse
            compose(reciprocal_real, b, n) = recip_seq(b, n)
        }
        seq_pointwise_eq_converges_to(compose(reciprocal_real, b), recip_seq(b),
            reciprocal_real(b_lim))
        converges_to(recip_seq(b), reciprocal_real(b_lim))
        reciprocal_real_apply(b_lim)
        reciprocal_real(b_lim) = b_lim.inverse
        converges_to(recip_seq(b), b_lim.inverse)
    }
}


/// The termwise quotient of two sequences.
define div_seq(a: Nat -> Real, b: Nat -> Real, n: Nat) -> Real {
    a(n) / b(n)
}

/// The termwise quotient is the product with the termwise reciprocal.
theorem div_seq_eq_prod_recip(a: Nat -> Real, b: Nat -> Real) {
    div_seq(a, b) = prod_seq(a, recip_seq(b))
} by {
    forall(n: Nat) {
        div_seq(a, b, n) = a(n) / b(n)
        a(n) / b(n) = a(n) * b(n).inverse
        recip_seq(b, n) = b(n).inverse
        prod_seq(a, recip_seq(b), n) = a(n) * recip_seq(b, n)
        prod_seq(a, recip_seq(b), n) = a(n) * b(n).inverse
        div_seq(a, b, n) = prod_seq(a, recip_seq(b), n)
    }
    function_extensionality(div_seq(a, b), prod_seq(a, recip_seq(b)))
    div_seq(a, b) = prod_seq(a, recip_seq(b))
}

/// A quotient of convergent sequences with nonzero limit converges to the
/// quotient of the limits.
theorem converges_to_div_seq(a: Nat -> Real, b: Nat -> Real, a_lim: Real, b_lim: Real) {
    converges_to(a, a_lim) and converges_to(b, b_lim) and b_lim != Real.0
    implies converges_to(div_seq(a, b), a_lim / b_lim)
} by {
    if converges_to(a, a_lim) and converges_to(b, b_lim) and b_lim != Real.0 {
        converges_to_imp_converges(a, a_lim)
        converges(a)
        converges_to_imp_converges(b, b_lim)
        converges(b)
        converges_to_recip_seq(b, b_lim)
        converges_to(recip_seq(b), b_lim.inverse)
        converges_to_imp_converges(recip_seq(b), b_lim.inverse)
        converges(recip_seq(b))
        converges_imp_converges_to(a)
        converges_to(a, limit(a))
        converges_to_unique(a, a_lim, limit(a))
        a_lim = limit(a)
        limit(a) = a_lim
        converges_imp_converges_to(recip_seq(b))
        converges_to(recip_seq(b), limit(recip_seq(b)))
        converges_to_unique(recip_seq(b), b_lim.inverse, limit(recip_seq(b)))
        b_lim.inverse = limit(recip_seq(b))
        limit(recip_seq(b)) = b_lim.inverse
        limit_prod_seq(a, recip_seq(b))
        converges_to(prod_seq(a, recip_seq(b)), limit(a) * limit(recip_seq(b)))
        limit(a) * limit(recip_seq(b)) = a_lim * b_lim.inverse
        converges_to(prod_seq(a, recip_seq(b)), a_lim * b_lim.inverse)
        div_seq_eq_prod_recip(a, b)
        div_seq(a, b) = prod_seq(a, recip_seq(b))
        forall(n: Nat) {
            prod_seq(a, recip_seq(b), n) = div_seq(a, b, n)
        }
        seq_pointwise_eq_converges_to(prod_seq(a, recip_seq(b)), div_seq(a, b),
            a_lim * b_lim.inverse)
        converges_to(div_seq(a, b), a_lim * b_lim.inverse)
        a_lim / b_lim = a_lim * b_lim.inverse
        converges_to(div_seq(a, b), a_lim / b_lim)
    }
}

// =============================================================================
// L'Hôpital's rule
// =============================================================================

/// The sequential form of L'Hôpital's rule for the 0/0 case: every sequence
/// approaching a with terms distinct from a, along which g never vanishes, is
/// mapped by f/g to a sequence converging to df(a)/dg(a).
theorem lhopital_derivative_ratio_seq_limit(
    f: Real -> Real, g: Real -> Real, df: Real -> Real, dg: Real -> Real,
    a: Real, q: Nat -> Real) {
    is_derivative_fn(f, df) and is_derivative_fn(g, dg) and
    f(a) = Real.0 and g(a) = Real.0 and dg(a) != Real.0 and
    (forall(x: Real) { x != a implies g(x) != Real.0 }) and
    (forall(n: Nat) { q(n) != a }) and converges_to(q, a)
    implies converges_to(compose(quotient_fn(f, g), q), df(a) / dg(a))
} by {
    if is_derivative_fn(f, df) and is_derivative_fn(g, dg) and
       f(a) = Real.0 and g(a) = Real.0 and dg(a) != Real.0 and
       (forall(x: Real) { x != a implies g(x) != Real.0 }) and
       (forall(n: Nat) { q(n) != a }) and converges_to(q, a) {
        is_derivative_fn_at(f, df, a)
        has_derivative_at(f, a, df(a))
        difference_quotient_seq_limit(f, a, df(a), q)
        converges_to(compose(difference_quotient(f, a), q), df(a))
        is_derivative_fn_at(g, dg, a)
        has_derivative_at(g, a, dg(a))
        difference_quotient_seq_limit(g, a, dg(a), q)
        converges_to(compose(difference_quotient(g, a), q), dg(a))
        dq_denominator_seq_nonzero(g, a, q)
        forall(n: Nat) {
            compose(difference_quotient(g, a), q, n) != Real.0
        }
        converges_to_div_seq(
            compose(difference_quotient(f, a), q),
            compose(difference_quotient(g, a), q), df(a), dg(a))
        converges_to(div_seq(
            compose(difference_quotient(f, a), q),
            compose(difference_quotient(g, a), q)), df(a) / dg(a))
        forall(n: Nat) {
            div_seq(
                compose(difference_quotient(f, a), q),
                compose(difference_quotient(g, a), q), n) =
                compose(difference_quotient(f, a), q, n) /
                compose(difference_quotient(g, a), q, n)
            compose(difference_quotient(f, a), q, n) =
                difference_quotient(f, a, q(n))
            compose(difference_quotient(g, a), q, n) =
                difference_quotient(g, a, q(n))
            f(a) = Real.0
            g(a) = Real.0
            forall(n0: Nat) { q(n0) != a }
            q(n) != a
            forall(x: Real) { x != a implies g(x) != Real.0 }
            q(n) != a implies g(q(n)) != Real.0
            g(q(n)) != Real.0
            lhopital_pointwise_dq_ratio(f, g, a, q(n))
            difference_quotient(f, a, q(n)) / difference_quotient(g, a, q(n)) =
                quotient_fn(f, g, q(n))
            compose(difference_quotient(f, a), q, n) /
                compose(difference_quotient(g, a), q, n) =
                quotient_fn(f, g, q(n))
            div_seq(
                compose(difference_quotient(f, a), q),
                compose(difference_quotient(g, a), q), n) =
                quotient_fn(f, g, q(n))
            compose(quotient_fn(f, g), q, n) = quotient_fn(f, g, q(n))
            div_seq(
                compose(difference_quotient(f, a), q),
                compose(difference_quotient(g, a), q), n) =
                compose(quotient_fn(f, g), q, n)
        }
        seq_pointwise_eq_converges_to(
            div_seq(
                compose(difference_quotient(f, a), q),
                compose(difference_quotient(g, a), q)),
            compose(quotient_fn(f, g), q), df(a) / dg(a))
        converges_to(compose(quotient_fn(f, g), q), df(a) / dg(a))
    }
}

/// L'Hôpital's rule, 0/0 case at a point with continuous derivatives: if f and g
/// are differentiable everywhere, vanish at a, g stays nonzero away from a, and
/// g'(a) is nonzero, then the limit of f/g at the punctured point a is
/// f'(a)/g'(a).  The proof only needs differentiability at a; the continuity
/// hypotheses on df and dg match the classical statement in which the
/// derivative ratio is evaluated directly at a.
theorem lhopital_continuous_derivatives(f: Real -> Real, g: Real -> Real,
    df: Real -> Real, dg: Real -> Real, a: Real) {
    continuous_at(df, a) and continuous_at(dg, a) and is_derivative_fn(f, df) and
    is_derivative_fn(g, dg) and f(a) = Real.0 and g(a) = Real.0 and
    dg(a) != Real.0 and (forall(x: Real) { x != a implies g(x) != Real.0 })
    implies limit_at(quotient_fn(f, g), a, df(a) / dg(a))
} by {
    if continuous_at(df, a) and continuous_at(dg, a) and is_derivative_fn(f, df) and
       is_derivative_fn(g, dg) and f(a) = Real.0 and g(a) = Real.0 and
       dg(a) != Real.0 and (forall(x: Real) { x != a implies g(x) != Real.0 }) {
        forall(q: Nat -> Real) {
            if (forall(n: Nat) { q(n) != a }) and converges_to(q, a) {
                lhopital_derivative_ratio_seq_limit(f, g, df, dg, a, q)
                converges_to(compose(quotient_fn(f, g), q), df(a) / dg(a))
            }
        }
        limit_at(quotient_fn(f, g), a, df(a) / dg(a)) = forall(q2: Nat -> Real) {
            (forall(n: Nat) { q2(n) != a }) and converges_to(q2, a)
            implies converges_to(compose(quotient_fn(f, g), q2), df(a) / dg(a))
        }
    }
}

/// L'Hôpital's rule, general 0/0 form: if the ratio of derivatives has limit l
/// at a, then the ratio of functions has limit l at a.
///
/// The proof would transfer the limit of df/dg along the Cauchy mean value
/// theorem witness c(x) between a and x (see `lhopital_ratio_cauchy_mvt`),
/// which requires sequential machinery for choosing and squeezing that witness;
/// it is left as future work.
// theorem lhopital_general(f: Real -> Real, g: Real -> Real, df: Real -> Real,
//     dg: Real -> Real, a: Real, l: Real) {
//     is_derivative_fn(f, df) and is_derivative_fn(g, dg) and
//     f(a) = Real.0 and g(a) = Real.0 and
//     (forall(x: Real) { x != a implies g(x) != Real.0 }) and
//     limit_at(quotient_fn(df, dg), a, l)
//     implies limit_at(quotient_fn(f, g), a, l)
// }
