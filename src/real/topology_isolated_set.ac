from data.basic.set import Set, intersection_contains_eq, intersection_contains_intro, set_ext,
    singleton_contains_eq, subset_contains
from real.real_field import Real
from real.topology import is_isolated_point_of_set
from real.topology_singleton import singleton_real_set_isolated

/// True if a real number is isolated in a given set.
define isolated_set_contains(s: Set[Real], x: Real) -> Bool {
    is_isolated_point_of_set(s, x)
}

/// The set of isolated points of a real set.
define isolated_set(s: Set[Real]) -> Set[Real] {
    Set[Real].new(isolated_set_contains(s))
}

/// Membership in the isolated set is isolated-point membership.
theorem isolated_set_contains_eq(s: Set[Real], x: Real) {
    isolated_set(s).contains(x) = is_isolated_point_of_set(s, x)
}

/// An isolated point belongs to the original real set.
theorem isolated_point_member(s: Set[Real], x: Real) {
    is_isolated_point_of_set(s, x) implies s.contains(x)
} by {
    if is_isolated_point_of_set(s, x) {
        is_isolated_point_of_set(s, x) = s.contains(x) and exists(eps: Real) {
            eps.is_positive and forall(y: Real) {
                s.contains(y) and y != x implies not y.is_close(x, eps)
            }
        }
        s.contains(x)
    }
}

/// A member of the isolated set belongs to the original set.
theorem isolated_set_member_in_set(s: Set[Real], x: Real) {
    isolated_set(s).contains(x) implies s.contains(x)
} by {
    if isolated_set(s).contains(x) {
        isolated_set_contains_eq(s, x)
        is_isolated_point_of_set(s, x)
        isolated_point_member(s, x)
        s.contains(x)
    }
}

/// The isolated set is contained in the original set.
theorem isolated_set_subset_self(s: Set[Real]) {
    isolated_set(s).subset(s)
} by {
    forall(x: Real) {
        if isolated_set(s).contains(x) {
            isolated_set_member_in_set(s, x)
        }
    }
}

/// An isolated point in a larger set is isolated in any subset that still contains it.
theorem isolated_point_of_subset(s: Set[Real], t: Set[Real], x: Real) {
    s.subset(t) and s.contains(x) and is_isolated_point_of_set(t, x) implies is_isolated_point_of_set(s, x)
} by {
    if s.subset(t) and s.contains(x) and is_isolated_point_of_set(t, x) {
        is_isolated_point_of_set(t, x) = t.contains(x) and exists(eps: Real) {
            eps.is_positive and forall(y: Real) {
                t.contains(y) and y != x implies not y.is_close(x, eps)
            }
        }
        let eps: Real satisfy {
            eps.is_positive and forall(y: Real) {
                t.contains(y) and y != x implies not y.is_close(x, eps)
            }
        }
        forall(y: Real) {
            if s.contains(y) and y != x {
                subset_contains(s, t, y)
                t.contains(y)
                not y.is_close(x, eps)
            }
        }
        exists(delta: Real) {
            delta.is_positive and forall(y: Real) {
                s.contains(y) and y != x implies not y.is_close(x, delta)
            }
        }
        is_isolated_point_of_set(s, x)
    }
}

/// A point isolated in the left set is isolated in the intersection when it lies in the right set.
theorem isolated_point_in_intersection_left(s: Set[Real], t: Set[Real], x: Real) {
    is_isolated_point_of_set(s, x) and t.contains(x)
    implies is_isolated_point_of_set(s.intersection(t), x)
} by {
    if is_isolated_point_of_set(s, x) and t.contains(x) {
        isolated_point_member(s, x)
        s.contains(x)
        intersection_contains_intro(s, t, x)
        s.intersection(t).contains(x)
        forall(y: Real) {
            if s.intersection(t).contains(y) {
                intersection_contains_eq(s, t, y)
                s.contains(y)
            }
        }
        s.intersection(t).subset(s)
        isolated_point_of_subset(s.intersection(t), s, x)
        is_isolated_point_of_set(s.intersection(t), x)
    }
}

/// A point isolated in the right set is isolated in the intersection when it lies in the left set.
theorem isolated_point_in_intersection_right(s: Set[Real], t: Set[Real], x: Real) {
    is_isolated_point_of_set(t, x) and s.contains(x)
    implies is_isolated_point_of_set(s.intersection(t), x)
} by {
    if is_isolated_point_of_set(t, x) and s.contains(x) {
        isolated_point_member(t, x)
        t.contains(x)
        intersection_contains_intro(s, t, x)
        s.intersection(t).contains(x)
        forall(y: Real) {
            if s.intersection(t).contains(y) {
                intersection_contains_eq(s, t, y)
                t.contains(y)
            }
        }
        s.intersection(t).subset(t)
        isolated_point_of_subset(s.intersection(t), t, x)
        is_isolated_point_of_set(s.intersection(t), x)
    }
}

/// The point of a singleton belongs to the isolated set of that singleton.
theorem singleton_isolated_set_contains_self(a: Real) {
    isolated_set(Set[Real].singleton(a)).contains(a)
} by {
    singleton_real_set_isolated(a)
    is_isolated_point_of_set(Set[Real].singleton(a), a)
    isolated_set_contains_eq(Set[Real].singleton(a), a)
    isolated_set(Set[Real].singleton(a)).contains(a)
}

/// The isolated set of a singleton is that singleton.
theorem isolated_set_singleton_eq_singleton(a: Real) {
    isolated_set(Set[Real].singleton(a)) = Set[Real].singleton(a)
} by {
    let left = isolated_set(Set[Real].singleton(a))
    let right = Set[Real].singleton(a)
    forall(x: Real) {
        if left.contains(x) {
            isolated_set_contains_eq(Set[Real].singleton(a), x)
            is_isolated_point_of_set(Set[Real].singleton(a), x)
            isolated_point_member(Set[Real].singleton(a), x)
            Set[Real].singleton(a).contains(x)
            right.contains(x)
        }
        if right.contains(x) {
            singleton_contains_eq[Real](a, x)
            x = a
            singleton_isolated_set_contains_self(a)
            left.contains(x)
        }
        left.contains(x) = right.contains(x)
    }
    set_ext(left, right)
}

/// The isolated set of a singleton is contained in the singleton.
theorem isolated_set_singleton_subset(a: Real) {
    isolated_set(Set[Real].singleton(a)).subset(Set[Real].singleton(a))
} by {
    isolated_set_singleton_eq_singleton(a)
}

/// The singleton is contained in its isolated set.
theorem singleton_subset_isolated_set_singleton(a: Real) {
    Set[Real].singleton(a).subset(isolated_set(Set[Real].singleton(a)))
} by {
    isolated_set_singleton_eq_singleton(a)
}

/// Intersecting with a set containing an isolated point preserves isolated-set membership.
theorem isolated_set_intersection_left_contains(s: Set[Real], t: Set[Real], x: Real) {
    isolated_set(s).contains(x) and t.contains(x) implies isolated_set(s.intersection(t)).contains(x)
} by {
    if isolated_set(s).contains(x) and t.contains(x) {
        isolated_set_contains_eq(s, x)
        is_isolated_point_of_set(s, x)
        isolated_point_in_intersection_left(s, t, x)
        is_isolated_point_of_set(s.intersection(t), x)
        isolated_set_contains_eq(s.intersection(t), x)
        isolated_set(s.intersection(t)).contains(x)
    }
}

/// Intersecting on the right with a set containing an isolated point preserves isolated-set membership.
theorem isolated_set_intersection_right_contains(s: Set[Real], t: Set[Real], x: Real) {
    isolated_set(t).contains(x) and s.contains(x) implies isolated_set(s.intersection(t)).contains(x)
} by {
    if isolated_set(t).contains(x) and s.contains(x) {
        isolated_set_contains_eq(t, x)
        is_isolated_point_of_set(t, x)
        isolated_point_in_intersection_right(s, t, x)
        is_isolated_point_of_set(s.intersection(t), x)
        isolated_set_contains_eq(s.intersection(t), x)
        isolated_set(s.intersection(t)).contains(x)
    }
}
