from order_set import closed_interval_set, left_open_interval_set, open_interval_set,
    right_open_interval_set
from data.basic.set import Set
from real.real_field import Real
from real.topology import empty_real_set_is_bounded, empty_set_is_connected, is_bounded_real_set,
    is_connected_real_set, singleton_real_set_is_bounded
from real.topology_bounded_algebra import intersection_of_bounded_real_sets_is_bounded
from real.topology_compact import compact_real_set_is_bounded, is_compact_real_set
from real.topology_connected_algebra import intersection_of_connected_real_sets_is_connected,
    singleton_real_set_is_connected
from real.topology_connected_intervals import closed_interval_set_is_connected,
    left_open_interval_set_is_connected, open_interval_set_is_connected,
    right_open_interval_set_is_connected
from real.topology_intervals import closed_interval_set_is_bounded, left_open_interval_set_is_bounded,
    open_interval_set_is_bounded, right_open_interval_set_is_bounded

/// True if a real set is bounded and connected.
define is_bounded_connected_real_set(s: Set[Real]) -> Bool {
    is_bounded_real_set(s) and is_connected_real_set(s)
}

/// A bounded connected real set is bounded.
theorem bounded_connected_real_set_is_bounded(s: Set[Real]) {
    is_bounded_connected_real_set(s) implies is_bounded_real_set(s)
} by {
    if is_bounded_connected_real_set(s) {
        is_bounded_connected_real_set(s) = (is_bounded_real_set(s) and is_connected_real_set(s))
        is_bounded_real_set(s)
    }
}

/// A bounded connected real set is connected.
theorem bounded_connected_real_set_is_connected(s: Set[Real]) {
    is_bounded_connected_real_set(s) implies is_connected_real_set(s)
} by {
    if is_bounded_connected_real_set(s) {
        is_bounded_connected_real_set(s) = (is_bounded_real_set(s) and is_connected_real_set(s))
        is_connected_real_set(s)
    }
}

/// A bounded and connected real set is bounded connected.
theorem bounded_connected_real_set_intro(s: Set[Real]) {
    is_bounded_real_set(s) and is_connected_real_set(s) implies is_bounded_connected_real_set(s)
}

/// The empty real set is bounded connected.
theorem empty_real_set_is_bounded_connected {
    is_bounded_connected_real_set(Set[Real].empty_set)
} by {
    empty_real_set_is_bounded
    empty_set_is_connected
    is_bounded_real_set(Set[Real].empty_set)
    is_connected_real_set(Set[Real].empty_set)
    is_bounded_connected_real_set(Set[Real].empty_set)
}

/// A singleton real set is bounded connected.
theorem singleton_real_set_is_bounded_connected(a: Real) {
    is_bounded_connected_real_set(Set[Real].singleton(a))
} by {
    singleton_real_set_is_bounded(a)
    singleton_real_set_is_connected(a)
    is_bounded_real_set(Set[Real].singleton(a))
    is_connected_real_set(Set[Real].singleton(a))
    is_bounded_connected_real_set(Set[Real].singleton(a))
}

/// The intersection of two bounded connected real sets is bounded connected.
theorem intersection_of_bounded_connected_real_sets_is_bounded_connected(s: Set[Real], t: Set[Real]) {
    is_bounded_connected_real_set(s) and is_bounded_connected_real_set(t)
    implies is_bounded_connected_real_set(s.intersection(t))
} by {
    if is_bounded_connected_real_set(s) and is_bounded_connected_real_set(t) {
        bounded_connected_real_set_is_bounded(s)
        bounded_connected_real_set_is_bounded(t)
        bounded_connected_real_set_is_connected(s)
        bounded_connected_real_set_is_connected(t)
        is_bounded_real_set(s)
        is_bounded_real_set(t)
        is_connected_real_set(s)
        is_connected_real_set(t)
        intersection_of_bounded_real_sets_is_bounded(s, t)
        intersection_of_connected_real_sets_is_connected(s, t)
        is_bounded_real_set(s.intersection(t))
        is_connected_real_set(s.intersection(t))
        is_bounded_connected_real_set(s.intersection(t))
    }
}

/// A compact connected real set is bounded connected.
theorem compact_connected_real_set_is_bounded_connected(s: Set[Real]) {
    is_compact_real_set(s) and is_connected_real_set(s) implies is_bounded_connected_real_set(s)
} by {
    if is_compact_real_set(s) and is_connected_real_set(s) {
        compact_real_set_is_bounded(s)
        is_bounded_real_set(s)
        is_bounded_connected_real_set(s)
    }
}

/// A closed interval is bounded connected.
theorem closed_interval_set_is_bounded_connected(lower: Real, upper: Real) {
    is_bounded_connected_real_set(closed_interval_set(lower, upper))
} by {
    closed_interval_set_is_bounded(lower, upper)
    closed_interval_set_is_connected(lower, upper)
    is_bounded_real_set(closed_interval_set(lower, upper))
    is_connected_real_set(closed_interval_set(lower, upper))
    is_bounded_connected_real_set(closed_interval_set(lower, upper))
}

/// An open interval is bounded connected.
theorem open_interval_set_is_bounded_connected(lower: Real, upper: Real) {
    is_bounded_connected_real_set(open_interval_set(lower, upper))
} by {
    open_interval_set_is_bounded(lower, upper)
    open_interval_set_is_connected(lower, upper)
    is_bounded_real_set(open_interval_set(lower, upper))
    is_connected_real_set(open_interval_set(lower, upper))
    is_bounded_connected_real_set(open_interval_set(lower, upper))
}

/// A left-open interval is bounded connected.
theorem left_open_interval_set_is_bounded_connected(lower: Real, upper: Real) {
    is_bounded_connected_real_set(left_open_interval_set(lower, upper))
} by {
    left_open_interval_set_is_bounded(lower, upper)
    left_open_interval_set_is_connected(lower, upper)
    is_bounded_real_set(left_open_interval_set(lower, upper))
    is_connected_real_set(left_open_interval_set(lower, upper))
    is_bounded_connected_real_set(left_open_interval_set(lower, upper))
}

/// A right-open interval is bounded connected.
theorem right_open_interval_set_is_bounded_connected(lower: Real, upper: Real) {
    is_bounded_connected_real_set(right_open_interval_set(lower, upper))
} by {
    right_open_interval_set_is_bounded(lower, upper)
    right_open_interval_set_is_connected(lower, upper)
    is_bounded_real_set(right_open_interval_set(lower, upper))
    is_connected_real_set(right_open_interval_set(lower, upper))
    is_bounded_connected_real_set(right_open_interval_set(lower, upper))
}
