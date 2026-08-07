from nat import Nat
from data.basic.set import Set
from real.real_field import Real
from real.sequence_cluster_sets import cluster_point_in_tail_closure,
    cluster_set_subset_closed_set_of_eventually_in, cluster_set_subset_closed_set_of_seq_in,
    sequence_cluster_set, sequence_cluster_set_contains_intro
from real.sequence_set_membership import seq_eventually_in_real_set, seq_in_real_set
from real.sequence_tail_sets import sequence_tail_set
from real.topology import adherent_point_eps, adherent_point_intro, closed_set_contains_adherent,
    closed_set_eq_closure, closure, eps_adherent_of_subset, is_adherent_point_of_set,
    is_bounded_real_set, is_closed_set, is_eps_adherent_to_set
from real.topology_closure import closure_is_closed
from real.topology_compact import closed_subset_of_compact_real_set_is_compact,
    compact_real_set_is_bounded, compact_real_set_is_closed, is_compact_real_set

/// The cluster set is contained in every tail closure.
theorem sequence_cluster_set_subset_tail_closure(a: Nat -> Real, m: Nat) {
    sequence_cluster_set(a).subset(closure(sequence_tail_set(a, m)))
} by {
    forall(x: Real) {
        if sequence_cluster_set(a).contains(x) {
            cluster_point_in_tail_closure(a, x, m)
            closure(sequence_tail_set(a, m)).contains(x)
        }
    }
}

/// A point adherent to the cluster set is adherent to every tail closure.
theorem cluster_set_adherent_to_tail_closure(a: Nat -> Real, x: Real, m: Nat) {
    is_adherent_point_of_set(sequence_cluster_set(a), x)
    implies is_adherent_point_of_set(closure(sequence_tail_set(a, m)), x)
} by {
    if is_adherent_point_of_set(sequence_cluster_set(a), x) {
        sequence_cluster_set_subset_tail_closure(a, m)
        sequence_cluster_set(a).subset(closure(sequence_tail_set(a, m)))
        forall(eps: Real) {
            if eps.is_positive {
                adherent_point_eps(sequence_cluster_set(a), x, eps)
                eps_adherent_of_subset(sequence_cluster_set(a), closure(sequence_tail_set(a, m)), x, eps)
                is_eps_adherent_to_set(closure(sequence_tail_set(a, m)), x, eps)
            }
        }
        adherent_point_intro(closure(sequence_tail_set(a, m)), x)
        is_adherent_point_of_set(closure(sequence_tail_set(a, m)), x)
    }
}

/// The cluster set of a real sequence is closed.
theorem sequence_cluster_set_is_closed(a: Nat -> Real) {
    is_closed_set(sequence_cluster_set(a))
} by {
    forall(x: Real) {
        if is_adherent_point_of_set(sequence_cluster_set(a), x) {
            forall(m: Nat) {
                cluster_set_adherent_to_tail_closure(a, x, m)
                is_adherent_point_of_set(closure(sequence_tail_set(a, m)), x)
                closure_is_closed(sequence_tail_set(a, m))
                is_closed_set(closure(sequence_tail_set(a, m)))
                closed_set_contains_adherent(closure(sequence_tail_set(a, m)), x)
                closure(sequence_tail_set(a, m)).contains(x)
            }
            sequence_cluster_set_contains_intro(a, x)
            sequence_cluster_set(a).contains(x)
        }
    }
}

/// The cluster set of a real sequence is equal to its closure.
theorem sequence_cluster_set_eq_closure(a: Nat -> Real) {
    sequence_cluster_set(a) = closure(sequence_cluster_set(a))
} by {
    sequence_cluster_set_is_closed(a)
    is_closed_set(sequence_cluster_set(a))
    closed_set_eq_closure(sequence_cluster_set(a))
    sequence_cluster_set(a) = closure(sequence_cluster_set(a))
}

/// A cluster set contained in a compact real set is compact.
theorem sequence_cluster_set_is_compact_of_subset_compact(a: Nat -> Real, s: Set[Real]) {
    sequence_cluster_set(a).subset(s) and is_compact_real_set(s)
    implies is_compact_real_set(sequence_cluster_set(a))
} by {
    if sequence_cluster_set(a).subset(s) and is_compact_real_set(s) {
        sequence_cluster_set_is_closed(a)
        is_closed_set(sequence_cluster_set(a))
        closed_subset_of_compact_real_set_is_compact(sequence_cluster_set(a), s)
        is_compact_real_set(sequence_cluster_set(a))
    }
}

/// A cluster set contained in a compact real set is bounded.
theorem sequence_cluster_set_is_bounded_of_subset_compact(a: Nat -> Real, s: Set[Real]) {
    sequence_cluster_set(a).subset(s) and is_compact_real_set(s)
    implies is_bounded_real_set(sequence_cluster_set(a))
} by {
    if sequence_cluster_set(a).subset(s) and is_compact_real_set(s) {
        sequence_cluster_set_is_compact_of_subset_compact(a, s)
        is_compact_real_set(sequence_cluster_set(a))
        compact_real_set_is_bounded(sequence_cluster_set(a))
        is_bounded_real_set(sequence_cluster_set(a))
    }
}

/// The cluster set of a sequence contained in a compact set is contained in that compact set.
theorem sequence_cluster_set_subset_compact_of_seq_in(a: Nat -> Real, s: Set[Real]) {
    seq_in_real_set(s, a) and is_compact_real_set(s) implies sequence_cluster_set(a).subset(s)
} by {
    if seq_in_real_set(s, a) and is_compact_real_set(s) {
        compact_real_set_is_closed(s)
        is_closed_set(s)
        cluster_set_subset_closed_set_of_seq_in(a, s)
        sequence_cluster_set(a).subset(s)
    }
}

/// The cluster set of a sequence eventually contained in a compact set is contained in that compact set.
theorem sequence_cluster_set_subset_compact_of_eventually_in(a: Nat -> Real, s: Set[Real]) {
    seq_eventually_in_real_set(s, a) and is_compact_real_set(s) implies sequence_cluster_set(a).subset(s)
} by {
    if seq_eventually_in_real_set(s, a) and is_compact_real_set(s) {
        compact_real_set_is_closed(s)
        is_closed_set(s)
        cluster_set_subset_closed_set_of_eventually_in(a, s)
        sequence_cluster_set(a).subset(s)
    }
}

/// The cluster set of a sequence contained in a compact set is compact.
theorem sequence_cluster_set_is_compact_of_seq_in_compact(a: Nat -> Real, s: Set[Real]) {
    seq_in_real_set(s, a) and is_compact_real_set(s) implies is_compact_real_set(sequence_cluster_set(a))
} by {
    if seq_in_real_set(s, a) and is_compact_real_set(s) {
        sequence_cluster_set_subset_compact_of_seq_in(a, s)
        sequence_cluster_set(a).subset(s)
        sequence_cluster_set_is_compact_of_subset_compact(a, s)
        is_compact_real_set(sequence_cluster_set(a))
    }
}

/// The cluster set of a sequence eventually contained in a compact set is compact.
theorem sequence_cluster_set_is_compact_of_eventually_in_compact(a: Nat -> Real, s: Set[Real]) {
    seq_eventually_in_real_set(s, a) and is_compact_real_set(s)
    implies is_compact_real_set(sequence_cluster_set(a))
} by {
    if seq_eventually_in_real_set(s, a) and is_compact_real_set(s) {
        sequence_cluster_set_subset_compact_of_eventually_in(a, s)
        sequence_cluster_set(a).subset(s)
        sequence_cluster_set_is_compact_of_subset_compact(a, s)
        is_compact_real_set(sequence_cluster_set(a))
    }
}

/// The cluster set of a sequence contained in a compact set is bounded.
theorem sequence_cluster_set_is_bounded_of_seq_in_compact(a: Nat -> Real, s: Set[Real]) {
    seq_in_real_set(s, a) and is_compact_real_set(s) implies is_bounded_real_set(sequence_cluster_set(a))
} by {
    if seq_in_real_set(s, a) and is_compact_real_set(s) {
        sequence_cluster_set_is_compact_of_seq_in_compact(a, s)
        is_compact_real_set(sequence_cluster_set(a))
        compact_real_set_is_bounded(sequence_cluster_set(a))
        is_bounded_real_set(sequence_cluster_set(a))
    }
}

/// The cluster set of a sequence eventually contained in a compact set is bounded.
theorem sequence_cluster_set_is_bounded_of_eventually_in_compact(a: Nat -> Real, s: Set[Real]) {
    seq_eventually_in_real_set(s, a) and is_compact_real_set(s)
    implies is_bounded_real_set(sequence_cluster_set(a))
} by {
    if seq_eventually_in_real_set(s, a) and is_compact_real_set(s) {
        sequence_cluster_set_is_compact_of_eventually_in_compact(a, s)
        is_compact_real_set(sequence_cluster_set(a))
        compact_real_set_is_bounded(sequence_cluster_set(a))
        is_bounded_real_set(sequence_cluster_set(a))
    }
}
