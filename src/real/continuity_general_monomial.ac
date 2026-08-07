from data.basic.functions import function_extensionality
from nat import Nat
from real.continuity_const_mul import const_mul_left, continuous_const_mul_left
from real.continuity_pointwise_pow import continuous_pointwise_pow, pointwise_pow
from real.continuity_base import Real, continuous

/// The function obtained by raising f(x) to the natural-number power n and multiplying by the constant c.
define general_monomial_fn(c: Real, f: Real -> Real, n: Nat, x: Real) -> Real {
    c * f(x).pow(n)
}

/// A general monomial agrees with the coefficient multiplied on the left of the pointwise power.
theorem general_monomial_fn_eq_const_mul_pointwise_pow(c: Real, f: Real -> Real, n: Nat) {
    general_monomial_fn(c, f, n) = const_mul_left(c, pointwise_pow(f, n))
} by {
    forall(x: Real) {
        general_monomial_fn(c, f, n, x) = c * f(x).pow(n)
        pointwise_pow(f, n, x) = f(x).pow(n)
        const_mul_left(c, pointwise_pow(f, n), x) = c * pointwise_pow(f, n, x)
        const_mul_left(c, pointwise_pow(f, n), x) = c * f(x).pow(n)
        general_monomial_fn(c, f, n, x) = const_mul_left(c, pointwise_pow(f, n), x)
    }
    function_extensionality(general_monomial_fn(c, f, n), const_mul_left(c, pointwise_pow(f, n)))
}

/// A constant times a natural-number power of a continuous real function is continuous.
theorem continuous_general_monomial_fn(c: Real, f: Real -> Real, n: Nat) {
    continuous(f) implies continuous(general_monomial_fn(c, f, n))
} by {
    if continuous(f) {
        continuous_pointwise_pow(f, n)
        continuous(pointwise_pow(f, n))
        continuous_const_mul_left(c, pointwise_pow(f, n))
        continuous(const_mul_left(c, pointwise_pow(f, n)))
        general_monomial_fn_eq_const_mul_pointwise_pow(c, f, n)
        continuous(general_monomial_fn(c, f, n))
    }
}
