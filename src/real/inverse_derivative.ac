/// The derivative of the inverse of a strictly increasing function.
/// /// If f is continuous and strictly increasing on [a, b], differentiable at an
/// interior point x0 with f'(x0) != 0, then the inverse function is
/// differentiable at f(x0) with derivative 1 / f'(x0).

from order import lte_trans, lt_trans, lt_imp_lte, not_lt_imp_gte, lte_antisymm, lt_of_lt_of_lte, not_lt_self
from order_set import closed_interval_set, closed_interval_set_contains_eq, closed_interval_set_lower_le, closed_interval_set_le_upper, closed_interval_set_contains_lower, closed_interval_set_contains_upper
from order import closed_interval
from real.continuity_base import Real, continuous
from real.derivative_basic import has_derivative_at, difference_quotient, has_derivative_at_delta, sub_ne_zero_of_ne
from real.calculus_api import is_derivative_fn, is_derivative_fn_at
from real.monotone_theorems import increasing_on, increasing_on_apply, f_inv_of, f_inv_spec, f_inv_left_inverse, f_inv_between_trap, increasing_on_values_imp_args_eq
from real.monotone_base import closed_interval_set_contains_of_bounds
from real.real_base import lt_add_pos, lt_add_right, lt_add_converse, pos_imp_eq_abs, close_imp_bounds, bounds_imp_close, pos_gt_zero, gt_zero_imp_pos, add_zero_left, add_zero_right, abs_neg, sub_cancels, abs_gte_zero
from real.real_seq import eps_smaller_than_both, lt_imp_minus_pos, close_and_lt_imp_close, neq_imp_abs_diff_pos
from real.real_ring import mul_zero_left, real_mul_comm, mul_one_right, mul_abs
from real.exp import abs_div, abs_inverse
from real.real_field import mul_inverse, inverse_div
from real.mean_value import inverse_inverse
from algebra.field.field import inverse_dist
from real.real_ring import exists_small_mul_variant_2
from real.real_ring import lt_mul_pos_right
from ordered_field import inverse_of_positive_is_positive, inverse_of_negative_is_negative, mul_lt_mul_of_pos_right, multiply_inequality_with_nonnegative_element
from real.derivative_quotient import inverse_local_abs_bound, inverse_sub_inverse, reciprocal_error_abs_lt_bound
from real.real_base import lte_lt_trans

numerals Real

/// The absolute difference of reciprocals equals the absolute difference over
/// the product of the absolute values.
theorem reciprocal_diff_abs(a: Real, b: Real) {
    a != Real.0 and b != Real.0 implies
    (a.inverse - b.inverse).abs = (a - b).abs / (a.abs * b.abs)
} by {
    if a != Real.0 and b != Real.0 {
        inverse_sub_inverse(a, b)
        a.inverse - b.inverse = (b - a) / (a * b)
        abs_div(b - a, a * b)
        ((b - a) / (a * b)).abs = (b - a).abs / (a * b).abs
        b - a = -(a - b)
        abs_neg(a - b)
        (-(a - b)).abs = (a - b).abs
        (b - a).abs = (a - b).abs
        mul_abs(a, b)
        a.abs * b.abs = (a * b).abs
        (a * b).abs = a.abs * b.abs
        (a.inverse - b.inverse).abs = (a - b).abs / (a.abs * b.abs)
    }
}

/// The inverse function is continuous at f(x0) when x0 is an interior point
/// of [a, b]: the preimage of every point close to f(x0) is close to x0.
theorem f_inv_continuous_at_interior(f: Real -> Real, a: Real, b: Real, x0: Real, eps_c: Real) {
    continuous(f) and a < b and increasing_on(f, a, b) and
    closed_interval_set(a, b).contains(x0) and a < x0 and x0 < b and eps_c.is_positive
    implies exists(delta_c: Real) {
        delta_c.is_positive and
        delta_c < f(x0) - f(a) and delta_c < f(b) - f(x0) and
        forall(y1: Real) {
            y1.is_close(f(x0), delta_c) implies
            (f_inv_of(f, a, b, y1) - f_inv_of(f, a, b, f(x0))).abs < eps_c
        }
    }
} by {
    if continuous(f) and a < b and increasing_on(f, a, b) and
       closed_interval_set(a, b).contains(x0) and a < x0 and x0 < b and eps_c.is_positive {
        f_inv_left_inverse(f, a, b, x0)
        f_inv_of(f, a, b, f(x0)) = x0
        lt_imp_lte(a, b)
        a <= b
        closed_interval_set_contains_lower(a, b)
        closed_interval_set(a, b).contains(a)
        closed_interval_set_contains_upper(a, b)
        closed_interval_set(a, b).contains(b)
        increasing_on_apply(f, a, b, a, x0)
        f(a) < f(x0)
        increasing_on_apply(f, a, b, x0, b)
        f(x0) < f(b)
        lt_imp_minus_pos(a, x0)
        (x0 - a).is_positive
        lt_imp_minus_pos(x0, b)
        (b - x0).is_positive
        lt_imp_minus_pos(f(a), f(x0))
        (f(x0) - f(a)).is_positive
        lt_imp_minus_pos(f(x0), f(b))
        (f(b) - f(x0)).is_positive
        eps_smaller_than_both(eps_c, x0 - a)
        let eta1: Real satisfy {
            eta1.is_positive and eta1 < eps_c and eta1 < x0 - a
        }
        eps_smaller_than_both(eta1, b - x0)
        let eta: Real satisfy {
            eta.is_positive and eta < eta1 and eta < b - x0
        }
        lt_trans[Real](eta, eta1, eps_c)
        eta < eps_c
        lt_trans[Real](eta, eta1, x0 - a)
        eta < x0 - a
        lt_add_pos(x0 - eta, eta)
        eta.is_positive
        x0 - eta < (x0 - eta) + eta
        (x0 - eta) + eta = x0
        x0 - eta < x0
        lt_add_right(eta, x0 - a, a)
        a + eta < a + (x0 - a)
        a + (x0 - a) = x0
        a + eta < x0
        lt_add_right(a + eta, x0, -eta)
        (a + eta) + -eta < x0 + -eta
        sub_cancels(a, eta)
        (a + eta) + -eta = a
        a < x0 + -eta
        x0 + -eta = x0 - eta
        a < x0 - eta
        lt_imp_lte(a, x0 - eta)
        a <= x0 - eta
        lt_trans[Real](x0 - eta, x0, b)
        x0 - eta < b
        lt_imp_lte(x0 - eta, b)
        x0 - eta <= b
        closed_interval_set_contains_of_bounds(a, b, x0 - eta)
        closed_interval_set(a, b).contains(x0 - eta)
        lt_add_pos(x0, eta)
        eta.is_positive
        x0 < x0 + eta
        lt_add_right(eta, b - x0, x0)
        x0 + eta < x0 + (b - x0)
        x0 + (b - x0) = b
        x0 + eta < b
        lt_trans[Real](a, x0, x0 + eta)
        a < x0 + eta
        lt_imp_lte(a, x0 + eta)
        a <= x0 + eta
        lt_imp_lte(x0 + eta, b)
        x0 + eta <= b
        closed_interval_set_contains_of_bounds(a, b, x0 + eta)
        closed_interval_set(a, b).contains(x0 + eta)
        lt_trans[Real](x0 - eta, x0, x0 + eta)
        x0 - eta < x0 + eta
        increasing_on_apply(f, a, b, x0 - eta, x0)
        f(x0 - eta) < f(x0)
        increasing_on_apply(f, a, b, x0, x0 + eta)
        f(x0) < f(x0 + eta)
        lt_imp_minus_pos(f(x0 - eta), f(x0))
        (f(x0) - f(x0 - eta)).is_positive
        lt_imp_minus_pos(f(x0), f(x0 + eta))
        (f(x0 + eta) - f(x0)).is_positive
        eps_smaller_than_both(f(x0) - f(x0 - eta), f(x0 + eta) - f(x0))
        let delta1: Real satisfy {
            delta1.is_positive and
            delta1 < f(x0) - f(x0 - eta) and
            delta1 < f(x0 + eta) - f(x0)
        }
        eps_smaller_than_both(delta1, f(x0) - f(a))
        let delta2: Real satisfy {
            delta2.is_positive and delta2 < delta1 and delta2 < f(x0) - f(a)
        }
        eps_smaller_than_both(delta2, f(b) - f(x0))
        let delta: Real satisfy {
            delta.is_positive and delta < delta2 and delta < f(b) - f(x0)
        }
        lt_trans[Real](delta, delta2, delta1)
        delta < delta1
        lt_trans[Real](delta, delta2, f(x0) - f(a))
        delta < f(x0) - f(a)
        lt_trans[Real](delta, delta1, f(x0) - f(x0 - eta))
        delta < f(x0) - f(x0 - eta)
        lt_trans[Real](delta, delta1, f(x0 + eta) - f(x0))
        delta < f(x0 + eta) - f(x0)
        forall(y1: Real) {
            if y1.is_close(f(x0), delta) {
                close_imp_bounds(y1, f(x0), delta)
                y1 < f(x0) + delta
                f(x0) - delta < y1
                lt_add_right(delta, f(x0 + eta) - f(x0), f(x0))
                f(x0) + delta < f(x0) + (f(x0 + eta) - f(x0))
                f(x0) + (f(x0 + eta) - f(x0)) = f(x0 + eta)
                f(x0) + delta < f(x0 + eta)
                lt_trans[Real](y1, f(x0) + delta, f(x0 + eta))
                y1 < f(x0 + eta)
                lt_add_right(delta, f(x0) - f(x0 - eta), f(x0 - eta))
                f(x0 - eta) + delta < f(x0 - eta) + (f(x0) - f(x0 - eta))
                f(x0 - eta) + (f(x0) - f(x0 - eta)) = f(x0)
                f(x0 - eta) + delta < f(x0)
                lt_trans[Real](f(x0 - eta) + delta, f(x0), y1 + delta)
                f(x0 - eta) + delta < y1 + delta
                lt_add_converse(f(x0 - eta), y1, delta)
                f(x0 - eta) < y1
                lt_add_right(delta, f(x0) - f(a), f(a))
                f(a) + delta < f(a) + (f(x0) - f(a))
                f(a) + (f(x0) - f(a)) = f(x0)
                f(a) + delta < f(x0)
                lt_trans[Real](f(a) + delta, f(x0), y1 + delta)
                f(a) + delta < y1 + delta
                lt_add_converse(f(a), y1, delta)
                f(a) < y1
                lt_imp_lte(f(a), y1)
                f(a) <= y1
                lt_add_right(delta, f(b) - f(x0), f(x0))
                f(x0) + delta < f(x0) + (f(b) - f(x0))
                f(x0) + (f(b) - f(x0)) = f(b)
                f(x0) + delta < f(b)
                lt_trans[Real](y1, f(x0) + delta, f(b))
                y1 < f(b)
                lt_imp_lte(y1, f(b))
                y1 <= f(b)
                closed_interval(f(a), f(b), y1)
                closed_interval_set_contains_eq(f(a), f(b), y1)
                closed_interval_set(f(a), f(b)).contains(y1)
                f_inv_between_trap(f, a, b, x0 - eta, x0 + eta, y1)
                x0 - eta < f_inv_of(f, a, b, y1)
                f_inv_of(f, a, b, y1) < x0 + eta
                f_inv_of(f, a, b, f(x0)) = x0
                bounds_imp_close(f_inv_of(f, a, b, y1), f_inv_of(f, a, b, f(x0)), eta)
                f_inv_of(f, a, b, y1).is_close(f_inv_of(f, a, b, f(x0)), eta)
                close_and_lt_imp_close(f_inv_of(f, a, b, y1), f_inv_of(f, a, b, f(x0)), eta, eps_c)
                eta < eps_c
                f_inv_of(f, a, b, y1).is_close(f_inv_of(f, a, b, f(x0)), eps_c)
                (f_inv_of(f, a, b, y1) - f_inv_of(f, a, b, f(x0))).abs < eps_c
            }
        }
        delta.is_positive and
        delta < f(x0) - f(a) and delta < f(b) - f(x0) and
        forall(y1: Real) {
            y1.is_close(f(x0), delta) implies
            (f_inv_of(f, a, b, y1) - f_inv_of(f, a, b, f(x0))).abs < eps_c
        }
        exists(delta_c: Real) {
            delta_c.is_positive and
            delta_c < f(x0) - f(a) and delta_c < f(b) - f(x0) and
            forall(y1: Real) {
                y1.is_close(f(x0), delta_c) implies
                (f_inv_of(f, a, b, y1) - f_inv_of(f, a, b, f(x0))).abs < eps_c
            }
        }
    }
}

/// The derivative of the inverse is the reciprocal derivative.
theorem inverse_derivative_at(f: Real -> Real, df: Real -> Real, a: Real, b: Real, x0: Real) {
    continuous(f) and a < b and is_derivative_fn(f, df) and
    increasing_on(f, a, b) and
    closed_interval_set(a, b).contains(x0) and a < x0 and x0 < b and
    df(x0) != Real.0
    implies has_derivative_at(f_inv_of(f, a, b), f(x0), Real.1 / df(x0))
} by {
    if continuous(f) and a < b and is_derivative_fn(f, df) and
       increasing_on(f, a, b) and
       closed_interval_set(a, b).contains(x0) and a < x0 and x0 < b and
       df(x0) != Real.0 {
        forall(eps: Real) {
            if eps.is_positive {
                is_derivative_fn_at(f, df, x0)
                has_derivative_at(f, x0, df(x0))
        inverse_local_abs_bound(df(x0))
        let (inv_delta: Real, bound: Real) satisfy {
            inv_delta.is_positive and bound.is_positive and forall(q: Real) {
                q.is_close(df(x0), inv_delta) implies q != Real.0 and q.inverse.abs <= bound
            }
        }
        neq_imp_abs_diff_pos(df(x0), Real.0)
        (df(x0) - Real.0).abs.is_positive
        df(x0) - Real.0 = df(x0)
        df(x0).abs.is_positive
        inverse_of_positive_is_positive[Real](df(x0).abs)
        Real.0 < df(x0).abs.inverse
        gt_zero_imp_pos(df(x0).abs.inverse)
        df(x0).abs.inverse.is_positive
        let factor = bound * df(x0).abs.inverse
        Real.0 < bound
        df(x0).abs.inverse.is_positive
        mul_lt_mul_of_pos_right[Real](Real.0, bound, df(x0).abs.inverse)
        Real.0 * df(x0).abs.inverse < bound * df(x0).abs.inverse
        mul_zero_left(df(x0).abs.inverse)
        Real.0 * df(x0).abs.inverse = Real.0
        Real.0 < bound * df(x0).abs.inverse
        gt_zero_imp_pos(factor)
        factor.is_positive
        exists_small_mul_variant_2(factor, eps)
        let eps_q: Real satisfy {
            eps_q.is_positive and eps_q * factor < eps
        }
        eps_smaller_than_both(eps_q, inv_delta)
        let eps_q2: Real satisfy {
            eps_q2.is_positive and eps_q2 < eps_q and eps_q2 < inv_delta
        }
        lt_trans[Real](eps_q2, eps_q, inv_delta)
        eps_q2 < inv_delta
        mul_lt_mul_of_pos_right[Real](eps_q2, eps_q, factor)
        eps_q2 * factor < eps_q * factor
        lt_trans[Real](eps_q2 * factor, eps_q * factor, eps)
        eps_q2 * factor < eps
        has_derivative_at_delta(f, x0, df(x0), eps_q2)
        let delta_d: Real satisfy {
            delta_d.is_positive and forall(x: Real) {
                x != x0 and x.is_close(x0, delta_d)
                implies difference_quotient(f, x0, x).is_close(df(x0), eps_q2)
            }
        }
        f_inv_continuous_at_interior(f, a, b, x0, delta_d)
        let delta_c: Real satisfy {
            delta_c.is_positive and
            delta_c < f(x0) - f(a) and delta_c < f(b) - f(x0) and
            forall(y1: Real) {
                y1.is_close(f(x0), delta_c) implies
                (f_inv_of(f, a, b, y1) - f_inv_of(f, a, b, f(x0))).abs < delta_d
            }
        }
        forall(y: Real) {
            if y != f(x0) and y.is_close(f(x0), delta_c) {
                delta_c.is_positive and
                delta_c < f(x0) - f(a) and delta_c < f(b) - f(x0) and
                forall(y1: Real) {
                    y1.is_close(f(x0), delta_c) implies
                    (f_inv_of(f, a, b, y1) - f_inv_of(f, a, b, f(x0))).abs < delta_d
                }
                (f_inv_of(f, a, b, y) - f_inv_of(f, a, b, f(x0))).abs < delta_d
                f_inv_left_inverse(f, a, b, x0)
                f_inv_of(f, a, b, f(x0)) = x0
                (f_inv_of(f, a, b, y) - x0).abs < delta_d
                f_inv_of(f, a, b, y).is_close(x0, delta_d)
                close_imp_bounds(y, f(x0), delta_c)
                y < f(x0) + delta_c
                f(x0) - delta_c < y
                lt_add_right(delta_c, f(x0) - f(a), f(a))
                f(a) + delta_c < f(a) + (f(x0) - f(a))
                f(a) + (f(x0) - f(a)) = f(x0)
                f(a) + delta_c < f(x0)
                lt_add_right(f(a) + delta_c, f(x0), -delta_c)
                (f(a) + delta_c) + -delta_c < f(x0) + -delta_c
                sub_cancels(f(a), delta_c)
                (f(a) + delta_c) + -delta_c = f(a)
                f(a) < f(x0) + -delta_c
                f(x0) + -delta_c = f(x0) - delta_c
                f(a) < f(x0) - delta_c
                lt_trans[Real](f(a), f(x0) - delta_c, y)
                f(a) < y
                lt_imp_lte(f(a), y)
                f(a) <= y
                lt_add_right(delta_c, f(b) - f(x0), f(x0))
                f(x0) + delta_c < f(x0) + (f(b) - f(x0))
                f(x0) + (f(b) - f(x0)) = f(b)
                f(x0) + delta_c < f(b)
                y < f(x0) + delta_c
                lt_trans[Real](y, f(x0) + delta_c, f(b))
                y < f(b)
                lt_imp_lte(y, f(b))
                y <= f(b)
                closed_interval(f(a), f(b), y)
                closed_interval_set_contains_eq(f(a), f(b), y)
                closed_interval_set(f(a), f(b)).contains(y)
                f_inv_spec(f, a, b, y)
                f(f_inv_of(f, a, b, y)) = y
                f_inv_of(f, a, b, f(x0)) = x0
                f(f_inv_of(f, a, b, f(x0))) = f(x0)
                f_inv_of(f, a, b, y) != x0
                delta_d.is_positive and forall(x: Real) {
                    x != x0 and x.is_close(x0, delta_d)
                    implies difference_quotient(f, x0, x).is_close(df(x0), eps_q2)
                }
                difference_quotient(f, x0, f_inv_of(f, a, b, y)).is_close(df(x0), eps_q2)
                (difference_quotient(f, x0, f_inv_of(f, a, b, y)) - df(x0)).abs < eps_q2
                close_and_lt_imp_close(difference_quotient(f, x0, f_inv_of(f, a, b, y)), df(x0), eps_q2, inv_delta)
                difference_quotient(f, x0, f_inv_of(f, a, b, y)).is_close(df(x0), inv_delta)
                df(x0) != Real.0
                difference_quotient(f, x0, f_inv_of(f, a, b, y)) != Real.0
                let q = difference_quotient(f, x0, f_inv_of(f, a, b, y))
                reciprocal_diff_abs(q, df(x0))
                (q.inverse - df(x0).inverse).abs = (q - df(x0)).abs / (q.abs * df(x0).abs)
                abs_inverse(q)
                q.inverse.abs = q.abs.inverse
                abs_inverse(df(x0))
                df(x0).inverse.abs = df(x0).abs.inverse
                inverse_dist[Real](q.abs, df(x0).abs)
                (q.abs * df(x0).abs).inverse = q.abs.inverse * df(x0).abs.inverse
                (q - df(x0)).abs / (q.abs * df(x0).abs) = (q - df(x0)).abs * (q.abs * df(x0).abs).inverse
                (q - df(x0)).abs * (q.abs * df(x0).abs).inverse = (q - df(x0)).abs * q.abs.inverse * df(x0).abs.inverse
                (q.inverse - df(x0).inverse).abs = (q - df(x0)).abs * q.abs.inverse * df(x0).abs.inverse
                q.inverse.abs <= bound
                q.inverse.abs = q.abs.inverse
                q.abs.inverse <= bound
                abs_gte_zero(q - df(x0))
                Real.0 <= (q - df(x0)).abs
                abs_gte_zero(df(x0).inverse)
                Real.0 <= df(x0).inverse.abs
                multiply_inequality_with_nonnegative_element[Real](Real.0, (q - df(x0)).abs, df(x0).inverse.abs)
                Real.0 * df(x0).inverse.abs <= (q - df(x0)).abs * df(x0).inverse.abs
                mul_zero_left(df(x0).inverse.abs)
                Real.0 * df(x0).inverse.abs = Real.0
                Real.0 <= (q - df(x0)).abs * df(x0).inverse.abs
                multiply_inequality_with_nonnegative_element[Real](q.abs.inverse, bound, (q - df(x0)).abs * df(x0).inverse.abs)
                q.abs.inverse * ((q - df(x0)).abs * df(x0).inverse.abs) <= bound * ((q - df(x0)).abs * df(x0).inverse.abs)
                q.abs.inverse * ((q - df(x0)).abs * df(x0).inverse.abs) = (q - df(x0)).abs * q.abs.inverse * df(x0).abs.inverse
                (q - df(x0)).abs * q.abs.inverse * df(x0).abs.inverse <= bound * ((q - df(x0)).abs * df(x0).inverse.abs)
                bound * ((q - df(x0)).abs * df(x0).inverse.abs) = (q - df(x0)).abs * bound * df(x0).inverse.abs
                (q.inverse - df(x0).inverse).abs <= (q - df(x0)).abs * bound * df(x0).inverse.abs
                gt_zero_imp_pos(bound)
                bound.is_positive
                gt_zero_imp_pos(df(x0).inverse.abs)
                df(x0).inverse.abs.is_positive
                factor = bound * df(x0).abs.inverse
                df(x0).inverse.abs = df(x0).abs.inverse
                bound * df(x0).inverse.abs = factor
                factor.is_positive
                Real.0 < factor
                (q - df(x0)).abs < eps_q2
                mul_lt_mul_of_pos_right[Real]((q - df(x0)).abs, eps_q2, bound * df(x0).inverse.abs)
                (q - df(x0)).abs * (bound * df(x0).inverse.abs) < eps_q2 * (bound * df(x0).inverse.abs)
                (q - df(x0)).abs * (bound * df(x0).inverse.abs) = (q - df(x0)).abs * bound * df(x0).inverse.abs
                (q - df(x0)).abs * bound * df(x0).inverse.abs < eps_q2 * (bound * df(x0).inverse.abs)
                eps_q2 * (bound * df(x0).inverse.abs) = eps_q2 * factor
                factor = bound * df(x0).abs.inverse
                df(x0).inverse.abs = df(x0).abs.inverse
                bound * df(x0).inverse.abs = factor
                eps_q2 * (bound * df(x0).inverse.abs) = eps_q2 * factor
                eps_q2 * factor < eps
                lt_trans[Real]((q - df(x0)).abs * bound * df(x0).inverse.abs,
                    eps_q2 * (bound * df(x0).inverse.abs), eps)
                (q - df(x0)).abs * bound * df(x0).inverse.abs < eps
                lte_lt_trans((q.inverse - df(x0).inverse).abs,
                    (q - df(x0)).abs * bound * df(x0).inverse.abs, eps)
                (q.inverse - df(x0).inverse).abs < eps
                difference_quotient(f_inv_of(f, a, b), f(x0), y) = (f_inv_of(f, a, b, y) - f_inv_of(f, a, b, f(x0))) / (y - f(x0))
                f_inv_of(f, a, b, f(x0)) = x0
                f(f_inv_of(f, a, b, y)) = y
                f_inv_of(f, a, b, f(x0)) = x0
                f(f_inv_of(f, a, b, f(x0))) = f(x0)
                y - f(x0) = f(f_inv_of(f, a, b, y)) - f(f_inv_of(f, a, b, f(x0)))
                difference_quotient(f_inv_of(f, a, b), f(x0), y) = (f_inv_of(f, a, b, y) - x0) / (f(f_inv_of(f, a, b, y)) - f(x0))
                q = (f(f_inv_of(f, a, b, y)) - f(x0)) / (f_inv_of(f, a, b, y) - x0)
                sub_ne_zero_of_ne(f_inv_of(f, a, b, y), x0)
                f_inv_of(f, a, b, y) - x0 != Real.0
                y - f(x0) != Real.0
                f(f_inv_of(f, a, b, y)) - f(x0) = y - f(x0)
                f(f_inv_of(f, a, b, y)) - f(x0) != Real.0
                inverse_div(f_inv_of(f, a, b, y) - x0, f(f_inv_of(f, a, b, y)) - f(x0))
                ((f_inv_of(f, a, b, y) - x0) / (f(f_inv_of(f, a, b, y)) - f(x0))).inverse =
                    (f(f_inv_of(f, a, b, y)) - f(x0)) / (f_inv_of(f, a, b, y) - x0)
                ((f_inv_of(f, a, b, y) - x0) / (f(f_inv_of(f, a, b, y)) - f(x0))).inverse = q
                difference_quotient(f_inv_of(f, a, b), f(x0), y).inverse = q
                inverse_inverse(difference_quotient(f_inv_of(f, a, b), f(x0), y))
                difference_quotient(f_inv_of(f, a, b), f(x0), y).inverse.inverse =
                    difference_quotient(f_inv_of(f, a, b), f(x0), y)
                difference_quotient(f_inv_of(f, a, b), f(x0), y).inverse.inverse = q.inverse
                difference_quotient(f_inv_of(f, a, b), f(x0), y) = q.inverse
                (difference_quotient(f_inv_of(f, a, b), f(x0), y) - df(x0).inverse).abs < eps
                Real.1 / df(x0) = df(x0).inverse
                difference_quotient(f_inv_of(f, a, b), f(x0), y).is_close(Real.1 / df(x0), eps)
            }
        }
        delta_c.is_positive and forall(y: Real) {
            y != f(x0) and y.is_close(f(x0), delta_c)
            implies difference_quotient(f_inv_of(f, a, b), f(x0), y).is_close(Real.1 / df(x0), eps)
        }
        exists(delta2: Real) {
            delta2.is_positive and forall(y: Real) {
                y != f(x0) and y.is_close(f(x0), delta2)
                implies difference_quotient(f_inv_of(f, a, b), f(x0), y).is_close(Real.1 / df(x0), eps)
            }
        }
            }
        }
    }
}
