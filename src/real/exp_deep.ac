/// Deep properties of the real exponential function.
///
/// This file extends `Real.exp.ac`, `exp_inequalities.ac` and `exp_log_properties.ac`
/// with further identities, order-theoretic characterizations, polynomial lower
/// bounds for the exponential series, and an upper bound relating `x.exp` to
/// `1 + x * x.exp`.

from nat import Nat, from_nat, factorial_one, factorial_step, pow_one
from rat import Rat
from order import lt_trans, lt_of_lte_of_lt, not_lt_imp_gte, not_lte_imp_gt, not_gte_imp_lt, not_gt_imp_lte
from real.exp import Real, exp_add, exp_zero, exp_pos, exp_increasing, exp_term, exp_term_partial_converges, exp_term_zero_index, exp_term_one_index, exp_term_pos, factorial_pos, pow_suc, pow_pos, two, two_positive, inverse_pos, exp_gt_one_plus_x, exp_ge_one_plus_x_plus_x2_half, from_nat_two_eq, one_div_two_eq_half
from real.exp_inequalities import exp_monotone, exp_ge_one_plus_self, exp_ge_one_of_nonneg, exp_le_one_of_nonpos, exp_lt_one_of_neg
from real.exp_log_properties import exp_gt_one_plus_x_strict, x_lt_exp
from real.log import exp_injective, exp_ne_zero, exp_neg, exp_nat_mul
from real.log_exp_foundations import exp_sub, exp_mul_exp_neg
from real.real_ring import converges, limit, mul_nonneg, mul_pos_pos, lt_mul_pos_left, lte_mul_nonneg_right, mul_le_mul_nonneg, from_nat_is_from_rat, real_mul_comm
from real.real_series import is_lower_bound, nonneg_partial_increasing, is_increasing, increasing_convergent_bounded_by_limit, is_upper_bound, pow_nonneg, seq_lte, seq_lte_preserves_limit, partial_suc, abs_conv_imp_conv
from real.abs_conv import absolutely_converges
from real.real_base import lt_add_pos, lte_lt_trans, lte_add_right, lt_add_right, add_lte_add, lte_add_left, lt_lte_trans, one_half_plus_one_half
from real.harmonic import real_recip_antitone_pos
from real.derivative_exp_log import exp_shift_term, exp_shift_sum, exp_shift_term_abs_converges, exp_sub_one_series_sum, factorial_real_lte_suc
from list import partial

numerals Real
numerals Nat

// =====================================================================
// Exponential algebraic identities
// =====================================================================
/// The exponential of twice an argument is the square of the exponential.
theorem exp_double(x: Real) {
    (x + x).exp = x.exp * x.exp
} by {
    exp_add(x, x)
    (x + x).exp = x.exp * x.exp
}

/// The square of the exponential is the exponential of twice the argument.
theorem exp_pow_two(x: Real) {
    x.exp.pow(Nat.2) = (two * x).exp
} by {
    pow_suc(x.exp, Nat.1)
    x.exp.pow(Nat.1.suc) = x.exp * x.exp.pow(Nat.1)
    Nat.1.suc = Nat.2
    x.exp.pow(Nat.2) = x.exp * x.exp.pow(Nat.1)
    pow_one[Real](x.exp)
    x.exp.pow(Nat.1) = x.exp
    x.exp.pow(Nat.2) = x.exp * x.exp
    exp_add(x, x)
    (x + x).exp = x.exp * x.exp
    x.exp.pow(Nat.2) = (x + x).exp
    two * x = x + x
    x.exp.pow(Nat.2) = (two * x).exp
}

/// The exponential of two is the square of Euler's number.
theorem exp_two_e {
    two.exp = Real.e * Real.e
} by {
    exp_add(Real.1, Real.1)
    (Real.1 + Real.1).exp = (Real.1).exp * (Real.1).exp
    Real.1 + Real.1 = two
    two.exp = (Real.1).exp * (Real.1).exp
    Real.e = (Real.1).exp
    (Real.1).exp * (Real.1).exp = Real.e * Real.e
    two.exp = Real.e * Real.e
}

/// The square of Euler's number is the exponential of two.
theorem e_pow_two {
    Real.e.pow(Nat.2) = two.exp
} by {
    exp_pow_two(Real.1)
    (Real.1).exp.pow(Nat.2) = (two * Real.1).exp
    Real.e = (Real.1).exp
    Real.e.pow(Nat.2) = (two * Real.1).exp
    two * Real.1 = two
    Real.e.pow(Nat.2) = two.exp
}

/// Natural powers of Euler's number are exponentials of natural numbers.
theorem exp_e_pow_nat(n: Nat) {
    Real.e.pow(n) = (from_nat[Real](n)).exp
} by {
    exp_nat_mul(Real.1, n)
    (from_nat[Real](n) * Real.1).exp = (Real.1).exp.pow(n)
    from_nat[Real](n) * Real.1 = from_nat[Real](n)
    (from_nat[Real](n)).exp = (Real.1).exp.pow(n)
    Real.e = (Real.1).exp
    (Real.1).exp.pow(n) = Real.e.pow(n)
    (from_nat[Real](n)).exp = Real.e.pow(n)
    Real.e.pow(n) = (from_nat[Real](n)).exp
}

/// The exponential of a zero difference is one.
theorem exp_sub_self(x: Real) {
    (x - x).exp = Real.1
} by {
    exp_sub(x, x)
    (x - x).exp = x.exp / x.exp
    exp_pos(x)
    x.exp > Real.0
    x.exp != Real.0
    x.exp / x.exp = Real.1
    (x - x).exp = Real.1
}

/// The reciprocal of an exponential is the exponential of the negation.
theorem exp_recip(x: Real) {
    Real.1 / x.exp = (-x).exp
} by {
    exp_neg(x)
    (-x).exp = Real.1 / x.exp
    Real.1 / x.exp = (-x).exp
}

/// An exponential divided by itself is one.
theorem exp_div_self(x: Real) {
    x.exp / x.exp = Real.1
} by {
    exp_pos(x)
    x.exp > Real.0
    x.exp != Real.0
    x.exp / x.exp = Real.1
}

/// The product of two exponentials is positive.
theorem exp_pos_mul(x: Real, y: Real) {
    x.exp * y.exp > Real.0
} by {
    exp_pos(x)
    exp_pos(y)
    x.exp > Real.0
    y.exp > Real.0
    mul_pos_pos(x.exp, y.exp)
    x.exp * y.exp > Real.0
}

// =====================================================================
// Exponential order characterizations
// =====================================================================
/// Exponentials preserve the non-strict order in both directions.
theorem exp_le_iff_le(x: Real, y: Real) {
    x.exp <= y.exp iff x <= y
} by {
    if x.exp <= y.exp {
        if not x <= y {
            not_lte_imp_gt(x, y)
            x > y
            y < x
            exp_increasing(y, x)
            y.exp < x.exp
            false
        }
        x <= y
    }
    if x <= y {
        exp_monotone(x, y)
        x.exp <= y.exp
    }
    x.exp <= y.exp iff x <= y
}

/// Exponentials preserve the strict order in both directions.
theorem exp_lt_iff_lt(x: Real, y: Real) {
    x.exp < y.exp iff x < y
} by {
    if x.exp < y.exp {
        if not x < y {
            not_lt_imp_gte(x, y)
            x >= y
            y <= x
            exp_monotone(y, x)
            y.exp <= x.exp
            false
        }
        x < y
    }
    if x < y {
        exp_increasing(x, y)
        x.exp < y.exp
    }
    x.exp < y.exp iff x < y
}

/// Equal exponentials have equal arguments and conversely.
theorem exp_eq_iff_eq(x: Real, y: Real) {
    x.exp = y.exp iff x = y
} by {
    if x.exp = y.exp {
        exp_injective(x, y)
        x = y
    }
    if x = y {
        x.exp = y.exp
    }
    x.exp = y.exp iff x = y
}

/// The exponential exceeds one exactly on positive reals.
theorem exp_gt_one_iff(x: Real) {
    x.exp > Real.1 iff x > Real.0
} by {
    if x.exp > Real.1 {
        if not x > Real.0 {
            not_gt_imp_lte(x, Real.0)
            x <= Real.0
            exp_le_one_of_nonpos(x)
            x.exp <= Real.1
            false
        }
        x > Real.0
    }
    if x > Real.0 {
        exp_increasing(Real.0, x)
        (Real.0).exp < x.exp
        exp_zero
        (Real.0).exp = Real.1
        Real.1 < x.exp
        x.exp > Real.1
    }
    x.exp > Real.1 iff x > Real.0
}

/// The exponential is at least one exactly on nonnegative reals.
theorem exp_ge_one_iff(x: Real) {
    x.exp >= Real.1 iff x >= Real.0
} by {
    if x.exp >= Real.1 {
        if not x >= Real.0 {
            not_gte_imp_lt(x, Real.0)
            x < Real.0
            exp_lt_one_of_neg(x)
            x.exp < Real.1
            false
        }
        x >= Real.0
    }
    if x >= Real.0 {
        exp_ge_one_of_nonneg(x)
        x.exp >= Real.1
    }
    x.exp >= Real.1 iff x >= Real.0
}

/// The exponential is below one exactly on negative reals.
theorem exp_lt_one_iff(x: Real) {
    x.exp < Real.1 iff x < Real.0
} by {
    if x.exp < Real.1 {
        if not x < Real.0 {
            not_lt_imp_gte(x, Real.0)
            x >= Real.0
            exp_ge_one_of_nonneg(x)
            x.exp >= Real.1
            false
        }
        x < Real.0
    }
    if x < Real.0 {
        exp_lt_one_of_neg(x)
        x.exp < Real.1
    }
    x.exp < Real.1 iff x < Real.0
}

/// The exponential is at most one exactly on nonpositive reals.
theorem exp_le_one_iff(x: Real) {
    x.exp <= Real.1 iff x <= Real.0
} by {
    if x.exp <= Real.1 {
        if not x <= Real.0 {
            not_lte_imp_gt(x, Real.0)
            x > Real.0
            exp_increasing(Real.0, x)
            (Real.0).exp < x.exp
            exp_zero
            (Real.0).exp = Real.1
            Real.1 < x.exp
            false
        }
        x <= Real.0
    }
    if x <= Real.0 {
        exp_le_one_of_nonpos(x)
        x.exp <= Real.1
    }
    x.exp <= Real.1 iff x <= Real.0
}

// =====================================================================
// Polynomial lower bounds from the exponential series
// =====================================================================
/// Every partial sum of the exponential series is a lower bound for `x.exp`
/// when `x` is nonnegative.
theorem exp_ge_partial(x: Real, n: Nat) {
    x >= Real.0 implies partial(exp_term(x), n) <= x.exp
} by {
    if x >= Real.0 {
        x.exp = limit(partial(exp_term(x)))
        exp_term_partial_converges(x)
        converges(partial(exp_term(x)))
        forall(k: Nat) {
            pow_nonneg(x, k)
            Real.0 <= x.pow(k)
            factorial_pos(k)
            Real.from_rat(Rat.from_nat(k.factorial)) > Real.0
            exp_term(x, k) = x.pow(k) / Real.from_rat(Rat.from_nat(k.factorial))
            Real.from_rat(Rat.from_nat(k.factorial)) > Real.0
            Real.from_rat(Rat.from_nat(k.factorial)) != Real.0
            x.pow(k) / Real.from_rat(Rat.from_nat(k.factorial)) = x.pow(k) * Real.from_rat(Rat.from_nat(k.factorial)).inverse
            inverse_pos(Real.from_rat(Rat.from_nat(k.factorial)))
            Real.from_rat(Rat.from_nat(k.factorial)).inverse.is_positive
            Real.from_rat(Rat.from_nat(k.factorial)).inverse >= Real.0
            x.pow(k) >= Real.0
            Real.from_rat(Rat.from_nat(k.factorial)).inverse >= Real.0
            mul_nonneg(x.pow(k), Real.from_rat(Rat.from_nat(k.factorial)).inverse)
            x.pow(k) * Real.from_rat(Rat.from_nat(k.factorial)).inverse >= Real.0
            Real.0 <= x.pow(k) * Real.from_rat(Rat.from_nat(k.factorial)).inverse
            x.pow(k) / Real.from_rat(Rat.from_nat(k.factorial)) >= Real.0
            exp_term(x, k) >= Real.0
            Real.0 <= exp_term(x, k)
        }
        is_lower_bound(exp_term(x), Real.0)
        nonneg_partial_increasing(exp_term(x))
        is_increasing(partial(exp_term(x)))
        increasing_convergent_bounded_by_limit(partial(exp_term(x)))
        is_upper_bound(partial(exp_term(x)), limit(partial(exp_term(x))))
        partial(exp_term(x), n) <= limit(partial(exp_term(x)))
        partial(exp_term(x), n) <= x.exp
    }
}

/// The fourth partial sum of the exponential series.
theorem exp_partial_four(x: Real) {
    partial(exp_term(x), Nat.4) = Real.1 + x + x.pow(Nat.2) / two + x.pow(Nat.3) / Real.from_rat(Rat.from_nat(Nat.3.factorial))
} by {
    partial(exp_term(x), Nat.1) = exp_term(x, Nat.0)
    exp_term_zero_index(x)
    exp_term(x, Nat.0) = Real.1
    partial(exp_term(x), Nat.1) = Real.1
    partial(exp_term(x), Nat.2) = partial(exp_term(x), Nat.1) + exp_term(x, Nat.1)
    exp_term_one_index(x)
    exp_term(x, Nat.1) = x
    partial(exp_term(x), Nat.2) = Real.1 + x
    partial(exp_term(x), Nat.3) = partial(exp_term(x), Nat.2) + exp_term(x, Nat.2)
    exp_term(x, Nat.2) = x.pow(Nat.2) / Real.from_rat(Rat.from_nat(Nat.2.factorial))
    factorial_one
    Nat.1.factorial = Nat.1
    factorial_step(Nat.1)
    Nat.2.factorial = Nat.2 * Nat.1.factorial
    Nat.2.factorial = Nat.2 * Nat.1
    Nat.2 * Nat.1 = Nat.2
    Nat.2.factorial = Nat.2
    Real.from_rat(Rat.from_nat(Nat.2.factorial)) = from_nat[Real](Nat.2.factorial)
    from_nat[Real](Nat.2.factorial) = from_nat[Real](Nat.2)
    from_nat[Real](Nat.2) = two
    Real.from_rat(Rat.from_nat(Nat.2.factorial)) = two
    exp_term(x, Nat.2) = x.pow(Nat.2) / two
    partial(exp_term(x), Nat.3) = Real.1 + x + x.pow(Nat.2) / two
    partial(exp_term(x), Nat.4) = partial(exp_term(x), Nat.3) + exp_term(x, Nat.3)
    exp_term(x, Nat.3) = x.pow(Nat.3) / Real.from_rat(Rat.from_nat(Nat.3.factorial))
    partial(exp_term(x), Nat.4) = Real.1 + x + x.pow(Nat.2) / two + x.pow(Nat.3) / Real.from_rat(Rat.from_nat(Nat.3.factorial))
}

/// The fifth partial sum of the exponential series.
theorem exp_partial_five(x: Real) {
    partial(exp_term(x), Nat.5) = Real.1 + x + x.pow(Nat.2) / two + x.pow(Nat.3) / Real.from_rat(Rat.from_nat(Nat.3.factorial)) +
        x.pow(Nat.4) / Real.from_rat(Rat.from_nat(Nat.4.factorial))
} by {
    exp_partial_four(x)
    partial(exp_term(x), Nat.4) = Real.1 + x + x.pow(Nat.2) / two + x.pow(Nat.3) / Real.from_rat(Rat.from_nat(Nat.3.factorial))
    partial(exp_term(x), Nat.5) = partial(exp_term(x), Nat.4) + exp_term(x, Nat.4)
    exp_term(x, Nat.4) = x.pow(Nat.4) / Real.from_rat(Rat.from_nat(Nat.4.factorial))
    partial(exp_term(x), Nat.5) = Real.1 + x + x.pow(Nat.2) / two + x.pow(Nat.3) / Real.from_rat(Rat.from_nat(Nat.3.factorial)) +
        x.pow(Nat.4) / Real.from_rat(Rat.from_nat(Nat.4.factorial))
}

/// For nonnegative `x`, `x.exp` is at least the cubic lower bound.
theorem exp_ge_poly3(x: Real) {
    x >= Real.0 implies
        x.exp >= Real.1 + x + x.pow(Nat.2) / two + x.pow(Nat.3) / Real.from_rat(Rat.from_nat(Nat.3.factorial))
} by {
    if x >= Real.0 {
        exp_ge_partial(x, Nat.4)
        partial(exp_term(x), Nat.4) <= x.exp
        exp_partial_four(x)
        Real.1 + x + x.pow(Nat.2) / two + x.pow(Nat.3) / Real.from_rat(Rat.from_nat(Nat.3.factorial)) <= x.exp
        x.exp >= Real.1 + x + x.pow(Nat.2) / two + x.pow(Nat.3) / Real.from_rat(Rat.from_nat(Nat.3.factorial))
    }
}

/// For positive `x`, `x.exp` strictly exceeds the quadratic lower bound.
theorem exp_gt_poly2(x: Real) {
    x > Real.0 implies x.exp > Real.1 + x + x.pow(Nat.2) / two
} by {
    if x > Real.0 {
        exp_ge_partial(x, Nat.4)
        partial(exp_term(x), Nat.4) <= x.exp
        exp_partial_four(x)
        Real.1 + x + x.pow(Nat.2) / two + x.pow(Nat.3) / Real.from_rat(Rat.from_nat(Nat.3.factorial)) <= x.exp
        exp_term_pos(x, Nat.3)
        exp_term(x, Nat.3) > Real.0
        exp_term(x, Nat.3) = x.pow(Nat.3) / Real.from_rat(Rat.from_nat(Nat.3.factorial))
        x.pow(Nat.3) / Real.from_rat(Rat.from_nat(Nat.3.factorial)) > Real.0
        lt_add_pos(Real.1 + x + x.pow(Nat.2) / two, x.pow(Nat.3) / Real.from_rat(Rat.from_nat(Nat.3.factorial)))
        Real.1 + x + x.pow(Nat.2) / two < Real.1 + x + x.pow(Nat.2) / two + x.pow(Nat.3) / Real.from_rat(Rat.from_nat(Nat.3.factorial))
        lt_lte_trans(Real.1 + x + x.pow(Nat.2) / two, Real.1 + x + x.pow(Nat.2) / two + x.pow(Nat.3) / Real.from_rat(Rat.from_nat(Nat.3.factorial)),
            x.exp)
        Real.1 + x + x.pow(Nat.2) / two < x.exp
        x.exp > Real.1 + x + x.pow(Nat.2) / two
    }
}

/// For positive `x`, `x.exp` strictly exceeds the cubic lower bound.
theorem exp_gt_poly3(x: Real) {
    x > Real.0 implies
        x.exp > Real.1 + x + x.pow(Nat.2) / two + x.pow(Nat.3) / Real.from_rat(Rat.from_nat(Nat.3.factorial))
} by {
    if x > Real.0 {
        exp_ge_partial(x, Nat.5)
        partial(exp_term(x), Nat.5) <= x.exp
        exp_partial_five(x)
        Real.1 + x + x.pow(Nat.2) / two + x.pow(Nat.3) / Real.from_rat(Rat.from_nat(Nat.3.factorial)) + x.pow(Nat.4) / Real.from_rat(Rat.from_nat(Nat.4.factorial)) <= x.exp
        exp_term_pos(x, Nat.4)
        exp_term(x, Nat.4) > Real.0
        exp_term(x, Nat.4) = x.pow(Nat.4) / Real.from_rat(Rat.from_nat(Nat.4.factorial))
        x.pow(Nat.4) / Real.from_rat(Rat.from_nat(Nat.4.factorial)) > Real.0
        lt_add_pos(Real.1 + x + x.pow(Nat.2) / two + x.pow(Nat.3) / Real.from_rat(Rat.from_nat(Nat.3.factorial)), x.pow(Nat.4) / Real.from_rat(Rat.from_nat(Nat.4.factorial)))
        Real.1 + x + x.pow(Nat.2) / two + x.pow(Nat.3) / Real.from_rat(Rat.from_nat(Nat.3.factorial)) < Real.1 + x + x.pow(Nat.2) / two + x.pow(Nat.3) / Real.from_rat(Rat.from_nat(Nat.3.factorial)) +
            x.pow(Nat.4) / Real.from_rat(Rat.from_nat(Nat.4.factorial))
        lt_lte_trans(Real.1 + x + x.pow(Nat.2) / two + x.pow(Nat.3) / Real.from_rat(Rat.from_nat(Nat.3.factorial)), Real.1 + x + x.pow(Nat.2) / two + x.pow(Nat.3) / Real.from_rat(Rat.from_nat(Nat.3.factorial)) +
            x.pow(Nat.4) / Real.from_rat(Rat.from_nat(Nat.4.factorial)), x.exp)
        Real.1 + x + x.pow(Nat.2) / two + x.pow(Nat.3) / Real.from_rat(Rat.from_nat(Nat.3.factorial)) < x.exp
        x.exp > Real.1 + x + x.pow(Nat.2) / two + x.pow(Nat.3) / Real.from_rat(Rat.from_nat(Nat.3.factorial))
    }
}

/// For positive `x`, the exponential exceeds `x^2 / 2`.
theorem exp_gt_x2_half(x: Real) {
    x > Real.0 implies x.exp > x.pow(Nat.2) / two
} by {
    if x > Real.0 {
        exp_gt_poly2(x)
        x.exp > Real.1 + x + x.pow(Nat.2) / two
        Real.0 < Real.1
        lt_add_right(Real.0, Real.1, x)
        Real.0 + x < Real.1 + x
        Real.0 + x = x
        x < Real.1 + x
        lt_trans(Real.0, x, Real.1 + x)
        Real.0 < Real.1 + x
        lt_add_right(Real.0, Real.1 + x, x.pow(Nat.2) / two)
        Real.0 + x.pow(Nat.2) / two < Real.1 + x + x.pow(Nat.2) / two
        Real.0 + x.pow(Nat.2) / two = x.pow(Nat.2) / two
        x.pow(Nat.2) / two < Real.1 + x + x.pow(Nat.2) / two
        lt_trans(x.pow(Nat.2) / two, Real.1 + x + x.pow(Nat.2) / two, x.exp)
        x.pow(Nat.2) / two < x.exp
        x.exp > x.pow(Nat.2) / two
    }
}

// =====================================================================
// Exponential growth bounds
// =====================================================================
/// A shifted exponential term is at most the matching exponential term for
/// nonnegative arguments.
theorem exp_shift_term_le_exp_term_of_nonneg(x: Real, n: Nat) {
    x >= Real.0 implies exp_shift_term(x, n) <= exp_term(x, n)
} by {
    if x >= Real.0 {
        let denom_suc = Real.from_rat(Rat.from_nat(n.suc.factorial))
        let denom = Real.from_rat(Rat.from_nat(n.factorial))
        factorial_pos(n.suc)
        denom_suc > Real.0
        factorial_pos(n)
        denom > Real.0
        factorial_real_lte_suc(n)
        Real.from_rat(Rat.from_nat(n.factorial)) <= Real.from_rat(Rat.from_nat(n.suc.factorial))
        denom <= denom_suc
        real_recip_antitone_pos(denom, denom_suc)
        Real.1 / denom_suc <= Real.1 / denom
        pow_nonneg(x, n)
        Real.0 <= x.pow(n)
        lte_mul_nonneg_right(Real.1 / denom_suc, Real.1 / denom, x.pow(n))
        (Real.1 / denom_suc) * x.pow(n) <= (Real.1 / denom) * x.pow(n)
        x.pow(n) * (Real.1 / denom_suc) <= x.pow(n) * (Real.1 / denom)
        denom_suc != Real.0
        denom != Real.0
        Real.1 / denom_suc = denom_suc.inverse
        Real.1 / denom = denom.inverse
        exp_shift_term(x, n) = x.pow(n) / denom_suc
        exp_term(x, n) = x.pow(n) / denom
        x.pow(n) / denom_suc = x.pow(n) * (Real.1 / denom_suc)
        x.pow(n) / denom = x.pow(n) * (Real.1 / denom)
        exp_shift_term(x, n) <= exp_term(x, n)
    }
}

/// The shifted exponential series sum is at most `x.exp` for nonnegative `x`.
theorem exp_shift_sum_le_exp(x: Real) {
    x >= Real.0 implies exp_shift_sum(x) <= x.exp
} by {
    if x >= Real.0 {
        forall(n: Nat) {
            exp_shift_term_le_exp_term_of_nonneg(x, n)
            exp_shift_term(x, n) <= exp_term(x, n)
        }
        seq_lte(exp_shift_term(x), exp_term(x))
        exp_shift_term_abs_converges(x)
        absolutely_converges(exp_shift_term(x))
        abs_conv_imp_conv(exp_shift_term(x))
        converges(partial(exp_shift_term(x)))
        exp_term_partial_converges(x)
        converges(partial(exp_term(x)))
        seq_lte_preserves_limit(exp_shift_term(x), exp_term(x))
        limit(partial(exp_shift_term(x))) <= limit(partial(exp_term(x)))
        exp_shift_sum(x) = limit(partial(exp_shift_term(x)))
        x.exp = limit(partial(exp_term(x)))
        exp_shift_sum(x) <= x.exp
    }
}

/// For nonnegative `x`, `x.exp - 1` is at most `x * x.exp`.
theorem exp_le_one_plus_x_exp(x: Real) {
    x >= Real.0 implies x.exp <= Real.1 + x * x.exp
} by {
    if x >= Real.0 {
        exp_sub_one_series_sum(x)
        x.exp - Real.1 = x * exp_shift_sum(x)
        exp_shift_sum_le_exp(x)
        exp_shift_sum(x) <= x.exp
        not x.is_negative
        lte_mul_nonneg_right(exp_shift_sum(x), x.exp, x)
        x * exp_shift_sum(x) <= x * x.exp
        x.exp - Real.1 <= x * x.exp
        lte_add_right(x.exp - Real.1, x * x.exp, Real.1)
        x.exp - Real.1 + Real.1 <= x * x.exp + Real.1
        x.exp - Real.1 + Real.1 = x.exp
        x.exp <= Real.1 + x * x.exp
    }
}

/// The exponential of a natural-scaled argument is the natural power of the
/// exponential (commuted form).
theorem exp_nat_mul_comm(x: Real, n: Nat) {
    (x * from_nat[Real](n)).exp = x.exp.pow(n)
} by {
    real_mul_comm(x, from_nat[Real](n))
    x * from_nat[Real](n) = from_nat[Real](n) * x
    exp_nat_mul(x, n)
    (from_nat[Real](n) * x).exp = x.exp.pow(n)
    (x * from_nat[Real](n)).exp = x.exp.pow(n)
}

/// The square of the exponential of one half is Euler's number.
theorem exp_one_half_sq {
    (Real.one_half).exp * (Real.one_half).exp = Real.e
} by {
    exp_add(Real.one_half, Real.one_half)
    (Real.one_half + Real.one_half).exp = (Real.one_half).exp * (Real.one_half).exp
    one_half_plus_one_half
    Real.one_half + Real.one_half = Real.1
    (Real.1).exp = (Real.one_half).exp * (Real.one_half).exp
    Real.e = (Real.1).exp
    (Real.one_half).exp * (Real.one_half).exp = Real.e
}

/// The exponential of a natural number exceeds every fixed power of two
/// eventually (the powers of Euler's number diverge to infinity).
theorem exp_e_pow_nat_pos(n: Nat) {
    Real.e.pow(n) > Real.0
} by {
    exp_pos(Real.1)
    (Real.1).exp > Real.0
    Real.e = (Real.1).exp
    Real.e > Real.0
    pow_pos(Real.e, n)
    Real.e.pow(n) > Real.0
}
