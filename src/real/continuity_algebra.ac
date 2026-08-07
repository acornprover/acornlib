from data.basic.functions import compose
from real.continuity_base import Real, add_fns, continuous, continuous_at,
    continuous_condition
from real.real_seq import close_and_lt_imp_close

/// If for every positive eps there is a positive delta witness for the
/// continuity condition of f at x, then f is continuous at x.
theorem continuous_condition_imp_continuous_at(f: Real -> Real, x: Real) {
    forall(eps: Real) {
        eps.is_positive implies exists(delta: Real) {
            delta.is_positive and continuous_condition(f, x, delta, eps)
        }
    }
    implies
    continuous_at(f, x)
}

/// The composition of two functions, with the inner continuous at x and the
/// outer continuous at the inner image of x, is continuous at x.
theorem compose_continuous_at(f: Real -> Real, g: Real -> Real, x: Real) {
    continuous_at(g, x) and continuous_at(f, g(x))
    implies
    continuous_at(compose(f, g), x)
} by {
    if continuous_at(g, x) and continuous_at(f, g(x)) {
        let h: Real -> Real = compose(f, g)
        continuous_at(g, x) = forall(eps2: Real) {
            eps2.is_positive implies exists(delta: Real) {
                delta.is_positive and continuous_condition(g, x, delta, eps2)
            }
        }
        continuous_at(f, g(x)) = forall(eps2: Real) {
            eps2.is_positive implies exists(delta: Real) {
                delta.is_positive and continuous_condition(f, g(x), delta, eps2)
            }
        }
        forall(eps: Real) {
            if eps.is_positive {
                let delta1: Real satisfy {
                    delta1.is_positive and continuous_condition(f, g(x), delta1, eps)
                }
                let delta2: Real satisfy {
                    delta2.is_positive and continuous_condition(g, x, delta2, delta1)
                }
                continuous_condition(g, x, delta2, delta1) = forall(y: Real) {
                    y.is_close(x, delta2) implies g(y).is_close(g(x), delta1)
                }
                continuous_condition(f, g(x), delta1, eps) = forall(y: Real) {
                    y.is_close(g(x), delta1) implies f(y).is_close(f(g(x)), eps)
                }
                forall(x1: Real) {
                    if x1.is_close(x, delta2) {
                        g(x1).is_close(g(x), delta1)
                        f(g(x1)).is_close(f(g(x)), eps)
                        compose(f, g, x1) = f(g(x1))
                        compose(f, g, x) = f(g(x))
                        compose(f, g, x1).is_close(compose(f, g, x), eps)
                    }
                }
                continuous_condition(compose(f, g), x, delta2, eps)
                continuous_condition(h, x, delta2, eps) = continuous_condition(compose(f, g), x, delta2, eps)
                continuous_condition(h, x, delta2, eps)
                delta2.is_positive and continuous_condition(h, x, delta2, eps)
                exists(delta_inner: Real) {
                    delta_inner.is_positive and continuous_condition(h, x, delta_inner, eps)
                }
            }
        }
        continuous_condition_imp_continuous_at(h, x)
        continuous_at(h, x)
        continuous_at(compose(f, g), x)
    }
}

/// The composition of two continuous functions is continuous.
theorem compose_continuous(f: Real -> Real, g: Real -> Real) {
    continuous(f) and continuous(g)
    implies
    continuous(compose(f, g))
} by {
    forall(x: Real) {
        continuous_at(g, x)
        continuous_at(f, g(x))
        compose_continuous_at(f, g, x)
        continuous_at(compose(f, g), x)
    }
}

/// The pointwise sum of two functions continuous at x is continuous at x.
theorem add_fns_continuous_at(f: Real -> Real, g: Real -> Real, x: Real) {
    continuous_at(f, x) and continuous_at(g, x)
    implies
    continuous_at(add_fns(f, g), x)
} by {
    if continuous_at(f, x) and continuous_at(g, x) {
    continuous_at(f, x) = forall(eps2: Real) {
        eps2.is_positive implies exists(delta: Real) {
            delta.is_positive and continuous_condition(f, x, delta, eps2)
        }
    }
    continuous_at(g, x) = forall(eps2: Real) {
        eps2.is_positive implies exists(delta: Real) {
            delta.is_positive and continuous_condition(g, x, delta, eps2)
        }
    }
    forall(eps: Real) {
        if eps.is_positive {
            let eps2: Real satisfy {
                eps2.is_positive and eps2 + eps2 < eps
            }
            let delta1: Real satisfy {
                delta1.is_positive and continuous_condition(f, x, delta1, eps2)
            }
            let delta2: Real satisfy {
                delta2.is_positive and continuous_condition(g, x, delta2, eps2)
            }
            let delta: Real satisfy {
                delta.is_positive and delta < delta1 and delta < delta2
            }
            continuous_condition(f, x, delta1, eps2) = forall(y: Real) {
                y.is_close(x, delta1) implies f(y).is_close(f(x), eps2)
            }
            continuous_condition(g, x, delta2, eps2) = forall(y: Real) {
                y.is_close(x, delta2) implies g(y).is_close(g(x), eps2)
            }
            forall(x1: Real) {
                if x1.is_close(x, delta) {
                    close_and_lt_imp_close(x1, x, delta, delta1)
                    close_and_lt_imp_close(x1, x, delta, delta2)
                    x1.is_close(x, delta1)
                    x1.is_close(x, delta2)
                    f(x1).is_close(f(x), eps2)
                    g(x1).is_close(g(x), eps2)
                    (f(x1) + g(x1)).is_close(f(x) + g(x), eps2 + eps2)
                    close_and_lt_imp_close(f(x1) + g(x1), f(x) + g(x), eps2 + eps2, eps)
                    (f(x1) + g(x1)).is_close(f(x) + g(x), eps)
                    add_fns(f, g, x1) = f(x1) + g(x1)
                    add_fns(f, g, x) = f(x) + g(x)
                    add_fns(f, g, x1).is_close(add_fns(f, g, x), eps)
                }
            }
            continuous_condition(add_fns(f, g), x, delta, eps)
            delta.is_positive and continuous_condition(add_fns(f, g), x, delta, eps)
            exists(delta_inner: Real) {
                delta_inner.is_positive and continuous_condition(add_fns(f, g), x, delta_inner, eps)
            }
        }
    }
    continuous_condition_imp_continuous_at(add_fns(f, g), x)
    continuous_at(add_fns(f, g), x)
    }
}

/// The pointwise sum of two continuous functions is continuous.
theorem add_fns_continuous(f: Real -> Real, g: Real -> Real) {
    continuous(f) and continuous(g)
    implies
    continuous(add_fns(f, g))
} by {
    forall(x: Real) {
        continuous_at(f, x)
        continuous_at(g, x)
        add_fns_continuous_at(f, g, x)
        continuous_at(add_fns(f, g), x)
    }
}
