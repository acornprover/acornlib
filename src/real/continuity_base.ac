from nat import Nat, lte_trans, nat_lte_join_left, nat_lte_join_right
from rat import Rat, iop
from rat import smaller_positive
from data.basic.functions import compose, is_constant
from real.real_seq import cauchy_bound, seq_close, tail_bound, eps_smaller_than_both, rat_seq_is_close, eventual_eq, converges_imp_converges_to, close_and_lt_imp_close, smaller_rat_eps, neg_is_close
from real.real_ring import lift_seq, converges, converges_to, limit, rat_seq, limit_rat
from real.real_field import Real
from real.real_base import self_close, rat_intersect

// This file proves theorems about uniform convergence.

attributes Real {
    // Placeholder to let other modules import Real from here.
}

/// True if f satisfies the delta-epsilon condition for uniform continuity on rationals.
define rat_condition(f: Rat -> Real, delta: Rat, eps: Real) -> Bool {
    forall(r1: Rat, r2: Rat) {
        r1.is_close(r2, delta) implies f(r1).is_close(f(r2), eps)
    }
}

/// True if f is uniformly continuous on the rational numbers.
define rat_uniform(f: Rat -> Real) -> Bool {
    forall(eps: Real) {
        eps.is_positive implies exists(delta: Rat) {
            delta.is_positive and rat_condition(f, delta, eps)
        }
    }
}

// Uniform continuity implies sequential continuity.
theorem uniform_imp_seq(f: Rat -> Real, q: Nat -> Rat) {
    rat_uniform(f) and converges(lift_seq(q))
    implies
    converges(compose(f, q))
} by {
    forall(eps: Real) {
        if eps.is_positive {
            let delta: Rat satisfy {
                delta.is_positive and rat_condition(f, delta, eps)
            }
            let n: Nat satisfy {
                cauchy_bound(lift_seq(q), n, Real.from_rat(delta))
            }

            forall(i: Nat, j: Nat) {
                if n <= i and n <= j {
                    lift_seq(q, i) = Real.from_rat(q(i))
                    lift_seq(q, j) = Real.from_rat(q(j))
                    not cauchy_bound(lift_seq(q), n, Real.from_rat(delta)) or lift_seq(q, i).is_close(lift_seq(q, j), Real.from_rat(delta))
                    not Real.from_rat(q(i)).is_close(Real.from_rat(q(j)), Real.from_rat(delta)) or q(i).is_close(q(j), delta)
                    q(i).is_close(q(j), delta)
                    compose(f, q)(i).is_close(compose(f, q)(j), eps)
                }
            }

            exists(k0: Nat, k1: Nat) {
                n <= k0 and n <= k1 and
                not compose(f, q)(k0).is_close(compose(f, q)(k1), eps)
            } or cauchy_bound(compose(f, q), n, eps)
            cauchy_bound(compose(f, q), n, eps)
        }
    }
}

/// True if f satisfies the delta-epsilon condition for uniform continuity on reals.
define uniform_condition(f: Real -> Real, delta: Real, eps: Real) -> Bool {
    forall(r1: Real, r2: Real) {
        r1.is_close(r2, delta) implies f(r1).is_close(f(r2), eps)
    }
}

/// True if f is uniformly continuous on the real numbers.
define uniform(f: Real -> Real) -> Bool {
    forall(eps: Real) {
        eps.is_positive implies exists(delta: Real) {
            delta.is_positive and uniform_condition(f, delta, eps)
        }
    }
}

/// A uniformly continuous function has a modulus at every positive tolerance.
theorem uniform_witness(f: Real -> Real, eps: Real) {
    uniform(f) and eps.is_positive implies exists(delta: Real) {
        delta.is_positive and uniform_condition(f, delta, eps)
    }
} by {
    if eps.is_positive {
        let delta: Real satisfy {
            delta.is_positive and uniform_condition(f, delta, eps)
        }
        delta.is_positive and uniform_condition(f, delta, eps)
    }
}

theorem converges_imp_close_tail_bound(a: Nat -> Real, eps: Real) {
    converges(a) and eps.is_positive implies exists(n: Nat) {
        forall(i: Nat) {
            n <= i implies a(i).is_close(limit(a), eps)
        }
    }
} by {
    if converges(a) and eps.is_positive {
        converges(a)
        converges(a) implies converges_to(a, limit(a))
        converges_to(a, limit(a))
        forall(eps0: Real) {
            if eps0.is_positive {
                let n0: Nat satisfy {
                    tail_bound(a, limit(a), n0, eps0)
                }
                tail_bound(a, limit(a), n0, eps0)
                forall(i: Nat) {
                    n0 <= i implies a(i).is_close(limit(a), eps0)
                }
            }
        }
        converges_to(a, limit(a)) implies forall(eps0: Real) {
            eps0.is_positive implies exists(big_n3: Nat) {
                forall(i: Nat) {
                    big_n3 <= i implies a(i).is_close(limit(a), eps0)
                }
            }
        }
        forall(eps0: Real) {
            eps0.is_positive implies exists(big_n2: Nat) {
                forall(i: Nat) {
                    big_n2 <= i implies a(i).is_close(limit(a), eps0)
                }
            }
        }
        eps.is_positive implies exists(big_n1: Nat) {
            forall(i: Nat) {
                big_n1 <= i implies a(i).is_close(limit(a), eps)
            }
        }
        eps.is_positive
        exists(big_n1: Nat) {
            forall(i: Nat) {
                big_n1 <= i implies a(i).is_close(limit(a), eps)
            }
        }
        let big_n4: Nat satisfy {
            forall(i: Nat) {
                big_n4 <= i implies a(i).is_close(limit(a), eps)
            }
        }
        forall(i: Nat) {
            if big_n4 <= i {
                big_n4 <= i implies a(i).is_close(limit(a), eps)
                a(i).is_close(limit(a), eps)
            }
        }
        let n: Nat satisfy {
            tail_bound(a, limit(a), n, eps)
        }
        tail_bound(a, limit(a), n, eps)
        let big_n0: Nat satisfy {
            forall(i: Nat) {
                big_n0 <= i implies a(i).is_close(limit(a), eps)
            }
        }
        forall(i: Nat) {
            n <= i implies a(i).is_close(limit(a), eps)
        }
        forall(i: Nat) {
            big_n0 <= i implies a(i).is_close(limit(a), eps)
        }
        forall(i: Nat) {
            if big_n0 <= i {
                big_n0 <= i implies a(i).is_close(limit(a), eps)
                a(i).is_close(limit(a), eps)
            }
        }
        exists(big_n: Nat) {
            forall(i: Nat) {
                big_n <= i implies a(i).is_close(limit(a), eps)
            }
        }
        exists(n2: Nat) {
            tail_bound(a, limit(a), n2, eps)
        }
        exists(n3: Nat) {
            forall(i: Nat) {
                n3 <= i implies a(i).is_close(limit(a), eps)
            }
        }
    }
}

theorem uniform_condition_exists(g: Real -> Real, eps: Real, delta: Real) {
    eps.is_positive and delta.is_positive and uniform_condition(g, delta, eps)
    implies
    exists(delta2: Real) {
        delta2.is_positive and uniform_condition(g, delta2, eps)
    }
} by {
    if eps.is_positive and delta.is_positive and uniform_condition(g, delta, eps) {
        delta.is_positive and uniform_condition(g, delta, eps)
    }
}

theorem limit_compose_uniform(f: Real -> Real, a: Nat -> Real) {
    uniform(f) and converges(a)
    implies
    converges_to(compose(f, a), f(limit(a)))
} by {
    let fa = compose(f, a)
    forall(eps: Real) {
        if eps.is_positive {
            let delta: Real satisfy {
                delta.is_positive and uniform_condition(f, delta, eps)
            }
            let n: Nat satisfy {
                forall(i: Nat) {
                    n <= i implies a(i).is_close(limit(a), delta)
                }
            }

            forall(i: Nat) {
                if n <= i {
                    fa(i).is_close(f(limit(a)), eps)
                }
            }
            tail_bound(fa, f(limit(a)), n, eps)
        }
    }
}

theorem limit_compose_rat_uniform(f: Rat -> Real, a: Nat -> Rat, r: Rat) {
    rat_uniform(f)
    and converges_to(lift_seq(a), Real.from_rat(r))
    implies
    converges_to(compose(f, a), f(r))
} by {
    let fa = compose(f, a)
    forall(eps: Real) {
        if eps.is_positive {
            let delta: Rat satisfy {
                delta.is_positive and rat_condition(f, delta, eps)
            }
            let n: Nat satisfy {
                tail_bound(lift_seq(a), Real.from_rat(r), n, Real.from_rat(delta))
            }

            forall(i: Nat) {
                if n <= i {
                    let s = lift_seq(a)
                    compose(f, a, i) = fa(i)
                    compose(f, a, i) = f(a(i))
                    Real.from_rat(a(i)) = lift_seq(a, i)
                    lift_seq(a, i).is_close(Real.from_rat(r), Real.from_rat(delta))
                    not tail_bound(lift_seq(a), Real.from_rat(r), n, Real.from_rat(delta)) or not n <= i or lift_seq(a, i).is_close(Real.from_rat(r), Real.from_rat(delta))
                    not Real.from_rat(a(i)).is_close(Real.from_rat(r), Real.from_rat(delta)) or a(i).is_close(r, delta)
                    not a(i).is_close(r, delta) or not rat_condition(f, delta, eps) or f(a(i)).is_close(f(r), eps)
                    fa(i).is_close(f(r), eps)
                }
            }
            tail_bound(fa, f(r), n, eps)
        }
    }
}

/// Extends a uniformly continuous function on rationals to a function on reals.
define lift_uc(f: Rat -> Real, x: Real) -> Real {
    limit(compose(f, rat_seq(x)))
}

theorem lift_fixes_rats(f: Rat -> Real, r: Rat) {
    rat_uniform(f)
    implies
    lift_uc(f, Real.from_rat(r)) = f(r)
} by {
    converges_to(compose(f, rat_seq(Real.from_rat(r))), f(r))
}

theorem lift_uniform_is_uniform(f: Rat -> Real) {
    rat_uniform(f)
    implies
    uniform(lift_uc(f))
} by {
    forall(eps: Real) {
        if eps.is_positive {
            // We split epsilon into three because we have three approximations to combine:
            // f(a) is close to f(a_i) is close to f(b_i) is close to f(b).
            let eps3: Real satisfy {
                eps3.is_positive and eps3 + eps3 + eps3 < eps
            }

            let delta: Rat satisfy {
                delta.is_positive and rat_condition(f, delta, eps3)
            }

            forall(a: Real, b: Real) {
                if a.is_close(b, Real.from_rat(delta)) {
                    // f(a_i) is approximately f(a)
                    let fas = compose(f, rat_seq(a))
                    converges_to(fas, limit(fas))
                    let n1: Nat satisfy {
                        forall(i: Nat) {
                            n1 <= i implies
                            fas(i).is_close(limit(fas), eps3)
                        }
                    }

                    // f(b_i) is approximately f(b)
                    let fbs = compose(f, rat_seq(b))
                    converges_to(fbs, limit(fbs))
                    let n2: Nat satisfy {
                        forall(i: Nat) {
                            n2 <= i implies
                            fbs(i).is_close(limit(fbs), eps3)
                        }
                    }

                    // f(a_i) and f(b_i) are close
                    seq_close(lift_seq(rat_seq(a)), lift_seq(rat_seq(b)), Real.from_rat(delta))
                    let n3: Nat satisfy {
                        forall(i: Nat) {
                            n3 <= i implies
                            lift_seq(rat_seq(a))(i).is_close(lift_seq(rat_seq(b))(i), Real.from_rat(delta))
                        }
                    }

                    let n: Nat satisfy {
                        n1 <= n and n2 <= n and n3 <= n
                    }
                    lift_seq(rat_seq(a))(n).is_close(lift_seq(rat_seq(b))(n), Real.from_rat(delta))
                    Real.from_rat(rat_seq(a, n)) = lift_seq(rat_seq(a), n)
                    Real.from_rat(rat_seq(b, n)) = lift_seq(rat_seq(b), n)
                    not Real.from_rat(rat_seq(a, n)).is_close(Real.from_rat(rat_seq(b, n)), Real.from_rat(delta)) or rat_seq(a, n).is_close(rat_seq(b, n), delta)
                    not rat_seq(a, n).is_close(rat_seq(b, n), delta) or not rat_condition(f, delta, eps3) or f(rat_seq(a, n)).is_close(f(rat_seq(b, n)), eps3)
                    f(rat_seq(a)(n)).is_close(f(rat_seq(b)(n)), eps3)
                    fas(n).is_close(fbs(n), eps3)
                    limit(fas).is_close(fbs(n), eps3 + eps3)
                    limit(fas).is_close(limit(fbs), eps3 + eps3 + eps3)
                    limit(fas).is_close(limit(fbs), eps)

                    lift_uc(f)(a).is_close(lift_uc(f)(b), eps)
                }
            }

            exists(k0: Real, k1: Real) {
                k0.is_close(k1, Real.from_rat(delta)) and
                not lift_uc(f)(k0).is_close(lift_uc(f)(k1), eps)
            } or uniform_condition(lift_uc(f), Real.from_rat(delta), eps)
            uniform_condition(lift_uc(f), Real.from_rat(delta), eps)
            delta.is_positive and uniform_condition(lift_uc(f), Real.from_rat(delta), eps)
            exists(delta2: Real) {
                delta2.is_positive and uniform_condition(lift_uc(f), delta2, eps)
            }
        }
    }
}

theorem positive_real_has_close_rat(x: Real, delta: Real) {
    delta.is_positive implies exists(r: Rat) {
        Real.from_rat(r).is_close(x, delta)
    }
} by {
    if delta.is_positive {
        x.is_close(x, delta)
        x.is_close(x, delta) and x.is_close(Real.from_rat(rat_seq(x, Nat.zero)), Real.from_rat(iop(Nat.zero)))
        let r: Rat satisfy {
            Real.from_rat(r).is_close(x, delta) and Real.from_rat(r).is_close(Real.from_rat(rat_seq(x, Nat.zero)), Real.from_rat(iop(Nat.zero)))
        }
        Real.from_rat(r).is_close(x, delta)
    }
}

// When two uniform functions are not equal, they differ on some rational number.
theorem uniform_ne_imp_rat_ne(f: Real -> Real, g: Real -> Real) {
    uniform(f) and uniform(g) and f != g implies
    exists(r: Rat) {
        f(Real.from_rat(r)) != g(Real.from_rat(r))
    }
} by {
    let x: Real satisfy {
        f(x) != g(x)
    }
    let eps = (f(x) - g(x)).abs
    (f(x) - g(x)).abs.is_positive or g(x) = f(x)
    eps.is_positive
    let eps2: Real satisfy {
        eps2.is_positive and eps2 + eps2 < eps
    }

    let delta1: Real satisfy {
        delta1.is_positive and uniform_condition(f, delta1, eps2)
    }
    let delta2: Real satisfy {
        delta2.is_positive and uniform_condition(g, delta2, eps2)
    }
    let delta: Real satisfy {
        delta.is_positive and delta < delta1 and delta < delta2
    }

    let r: Rat satisfy {
        Real.from_rat(r).is_close(x, delta)
    }
    Real.from_rat(r).is_close(x, delta1)
    Real.from_rat(r).is_close(x, delta2)
    g(Real.from_rat(r)).is_close(g(x), eps2)
    if f(Real.from_rat(r)) = g(Real.from_rat(r)) {
        f(x).is_close(f(Real.from_rat(r)), eps2)
        f(x).is_close(g(x), eps2 + eps2)
    }
}

/// True if f is strictly increasing on the rational numbers.
define rat_increasing(f: Rat -> Real) -> Bool {
    forall(r1: Rat, r2: Rat) {
        r1 < r2 implies f(r1) < f(r2)
    }
}

/// True if f is strictly increasing on the real numbers.
define increasing(f: Real -> Real) -> Bool {
    forall(x: Real, y: Real) {
        x < y implies f(x) < f(y)
    }
}

theorem uni_cond_imp_rat_cond(f: Real -> Real, delta: Rat, eps: Real) {
    uniform_condition(f, Real.from_rat(delta), eps)
    implies
    rat_condition(compose(f, Real.from_rat), delta, eps)
}

theorem uniform_imp_rat_uniform(f: Real -> Real) {
    uniform(f)
    implies
    rat_uniform(compose(f, Real.from_rat))
} by {
    forall(eps: Real) {
        if eps.is_positive {
            let delta: Real satisfy {
                delta.is_positive and uniform_condition(f, delta, eps)
            }
            let rdelta: Rat satisfy {
                rdelta.is_positive and Real.from_rat(rdelta) < delta
            }
            forall(r1: Real, r2: Real) {
                if r1.is_close(r2, Real.from_rat(rdelta)) {
                    r1.is_close(r2, Real.from_rat(rdelta)) and Real.from_rat(rdelta) < delta
                    r1.is_close(r2, delta)
                    f(r1).is_close(f(r2), eps)
                }
            }
            exists(k0: Real, k1: Real) {
                k0.is_close(k1, Real.from_rat(rdelta)) and
                not f(k0).is_close(f(k1), eps)
            } or uniform_condition(f, Real.from_rat(rdelta), eps)
            uniform_condition(f, Real.from_rat(rdelta), eps)
            rat_condition(compose(f, Real.from_rat), rdelta, eps)
            rdelta.is_positive and rat_condition(compose(f, Real.from_rat), rdelta, eps)
        }
    }
}

theorem tighten_uc(f: Real -> Real, delta1: Real, delta2: Real, eps: Real) {
    delta1.is_positive and delta2.is_positive and
    uniform_condition(f, delta1, eps) and delta2 < delta1
    implies
    uniform_condition(f, delta2, eps)
} by {
    if delta1.is_positive and delta2.is_positive and uniform_condition(f, delta1, eps) and delta2 < delta1 {
        forall(r1: Real, r2: Real) {
            if r1.is_close(r2, delta2) {
                r1.is_close(r2, delta2) and delta2 < delta1
                r1.is_close(r2, delta1)
                f(r1).is_close(f(r2), eps)
            }
        }
    }
}

theorem add_real_eps_between(a: Real, b: Real) {
    a < b implies
    exists(eps: Real) {
        eps.is_positive and a + eps < b
    }
}

theorem lift_inc_is_inc(f: Rat -> Real) {
    rat_uniform(f) and rat_increasing(f)
    implies
    increasing(lift_uc(f))
} by {
    let lf = lift_uc(f)
    uniform(lf)
    forall(r: Rat) {
        lf(Real.from_rat(r)) = f(r)
    }
    forall(x: Real, y: Real) {
        if x < y {
            let r1: Rat satisfy {
                x < Real.from_rat(r1) and Real.from_rat(r1) < y
            }
            Real.from_rat(r1) < y
            let eps_r: Rat satisfy {
                eps_r.is_positive and Real.from_rat(r1) + Real.from_rat(eps_r) < y
            }
            Real.from_rat(r1) + Real.from_rat(eps_r) = Real.from_rat(r1 + eps_r)
            r1 < r1 + eps_r
            Real.from_rat(r1 + eps_r) < y
            let r2: Rat satisfy {
                r1 < r2 and Real.from_rat(r2) < y
            }
            f(r1) < f(r2)
            let eps: Real satisfy {
                eps.is_positive and f(r1) + eps < f(r2)
            }
            let eps2: Real satisfy {
                eps2.is_positive and eps2 + eps2 < eps
            }

            // Pick delta1 so that f(xr) is close to f(x)
            let delta1: Real satisfy {
                delta1.is_positive and
                uniform_condition(lf, delta1, eps2)
            }

            // Pick delta2 so that xr is close to x and yr is close to y
            let delta2a: Real satisfy {
                delta2a.is_positive and
                x + delta2a < Real.from_rat(r1)
            }
            let delta2b: Real satisfy {
                delta2b.is_positive and
                Real.from_rat(r2) + delta2b < y
            }
            let delta2: Real satisfy {
                delta2.is_positive and
                delta2 < delta2a and
                delta2 < delta2b
            }

            // Delta satisfies all these conditions
            let delta_min = delta1.min(delta2)
            delta_min.is_positive
            let delta: Real satisfy {
                delta.is_positive and
                delta < delta_min
            }
            delta < delta2
            delta < delta2a
            delta < delta2b

            let xr: Rat satisfy {
                Real.from_rat(xr).is_close(x, delta)
            }
            x + delta <= x + delta2a
            Real.from_rat(xr) < x + delta2a
            Real.from_rat(xr) < Real.from_rat(r1)

            let yr: Rat satisfy {
                Real.from_rat(yr).is_close(y, delta)
            }
            Real.from_rat(r2) + delta < Real.from_rat(r2) + delta2b
            Real.from_rat(r2) + delta < y
            not Real.from_rat(yr).is_close(y, delta) or y < Real.from_rat(yr) + delta
            y < Real.from_rat(yr) + delta
            y <= Real.from_rat(yr) + delta
            not y <= Real.from_rat(yr) + delta or not Real.from_rat(r2) + delta < y or Real.from_rat(r2) + delta < Real.from_rat(yr) + delta
            Real.from_rat(r2) + delta < Real.from_rat(yr) + delta
            Real.from_rat(r2) < Real.from_rat(yr)

            forall(z: Real, zr: Rat) {
                if Real.from_rat(zr).is_close(z, delta1) {
                    lf(Real.from_rat(zr)).is_close(lf(z), eps2)
                }
            }
            delta < delta1
            Real.from_rat(xr).is_close(x, delta1)
            Real.from_rat(yr).is_close(y, delta1)

            f(xr) < f(r1)
            f(r2) < f(yr)
            f(r2) <= f(yr)
            f(xr) + eps < f(r1) + eps
            not f(r2) <= f(yr) or not f(r1) + eps < f(r2) or f(r1) + eps < f(yr)
            f(r1) + eps < f(yr)
            not f(r1) + eps < f(yr) or f(r1) + eps <= f(yr)
            f(r1) + eps <= f(yr)
            f(xr) + eps < f(r1) + eps
            let aa = f(xr) + eps
            let bb = f(r1) + eps
            let cc = f(yr)
            aa < bb
            bb <= cc
            aa < bb and bb <= cc
            aa < cc
            f(xr) + eps < f(yr)

            lf(x) < f(xr) + eps2
            f(xr) + (eps2 + eps2) = f(xr) + eps2 + eps2
            not eps2 + eps2 < eps or f(xr) + (eps2 + eps2) < f(xr) + eps
            f(xr) + eps2 + eps2 < f(xr) + eps
            f(xr) + eps <= f(yr)
            let dd = f(xr) + eps2 + eps2
            let ee = f(xr) + eps
            let ff = f(yr)
            dd < ee
            ee <= ff
            dd < ee and ee <= ff
            dd < ff
            f(xr) + eps2 + eps2 < f(yr)
            lf(x) + eps2 < f(xr) + eps2 + eps2
            f(xr) + eps2 + eps2 <= f(yr)
            let gg = lf(x) + eps2
            let hh = f(xr) + eps2 + eps2
            let ii = f(yr)
            gg < hh
            hh <= ii
            gg < hh and hh <= ii
            gg < ii
            lf(x) + eps2 < f(yr)
            f(yr) < lf(y) + eps2
            f(yr) <= lf(y) + eps2
            let jj = lf(x) + eps2
            let kk = f(yr)
            let ll = lf(y) + eps2
            jj < kk
            kk <= ll
            jj < kk and kk <= ll
            jj < ll
            lf(x) + eps2 < lf(y) + eps2
            lf(x) < lf(y)
        }
    }
}

theorem constant_has_rat_uniform_witness(f: Rat -> Real, eps: Real) {
    is_constant(f) and eps.is_positive implies exists(delta: Rat) {
        delta.is_positive and rat_condition(f, delta, eps)
    }
} by {
    if is_constant(f) and eps.is_positive {
        Rat.1.is_positive
        let y: Real satisfy {
            forall(x: Rat) {
                f(x) = y
            }
        }
        forall(r1: Rat, r2: Rat) {
            if r1.is_close(r2, Rat.1) {
                f(r1) = y
                f(r2) = y
                f(r1).is_close(f(r2), eps)
            }
        }
        Rat.1.is_positive and rat_condition(f, Rat.1, eps)
    }
}

theorem constant_is_rat_uniform(f: Rat -> Real) {
    is_constant(f)
    implies
    rat_uniform(f)
} by {
    forall(eps: Real) {
        if eps.is_positive {
            exists(delta: Rat) {
                delta.is_positive and rat_condition(f, delta, eps)
            }
        }
    }
}

theorem constant_has_uniform_witness(f: Real -> Real, eps: Real) {
    is_constant(f) and eps.is_positive implies exists(delta: Real) {
        delta.is_positive and uniform_condition(f, delta, eps)
    }
} by {
    if is_constant(f) and eps.is_positive {
        let y: Real satisfy {
            forall(x: Real) {
                f(x) = y
            }
        }
        forall(r1: Real, r2: Real) {
            if r1.is_close(r2, eps) {
                f(r1) = y
                f(r2) = y
                f(r1).is_close(f(r2), eps)
            }
        }
        eps.is_positive and uniform_condition(f, eps, eps)
    }
}

theorem constant_is_uniform(f: Real -> Real) {
    is_constant(f)
    implies
    uniform(f)
} by {
    forall(eps: Real) {
        if eps.is_positive {
            exists(delta: Real) {
                delta.is_positive and uniform_condition(f, delta, eps)
            }
        }
    }
}

theorem lift_constant_is_constant(f: Rat -> Real) {
    is_constant(f)
    implies
    is_constant(lift_uc(f))
} by {
    let (y: Real) satisfy {
        forall(r: Rat) {
            f(r) = y
        }
    }
    forall(x: Real) {
        forall(n: Nat) {
            compose(f, rat_seq(x))(n) = y
        }
        eventual_eq(compose(f, rat_seq(x)), y)
        lift_uc(f)(x) = y
    }
    exists(y0: Real) {
        forall(x: Real) {
            lift_uc(f)(x) = y0
        }
    }
}

theorem compose_uniform(f: Real -> Real, g: Real -> Real) {
    uniform(f) and uniform(g)
    implies
    uniform(compose(f, g))
} by {
    forall(eps: Real) {
        if eps.is_positive {
            let delta1: Real satisfy {
                delta1.is_positive and uniform_condition(f, delta1, eps)
            }
            let delta2: Real satisfy {
                delta2.is_positive and uniform_condition(g, delta2, delta1)
            }

            forall(x: Real, y: Real) {
                if x.is_close(y, delta2) {
                    compose(f, g)(x).is_close(compose(f, g)(y), eps)
                }
            }

            exists(k0: Real, k1: Real) {
                k0.is_close(k1, delta2) and
                not compose(f, g)(k0).is_close(compose(f, g)(k1), eps)
            } or uniform_condition(compose(f, g), delta2, eps)
            uniform_condition(compose(f, g), delta2, eps)
            delta2.is_positive and uniform_condition(compose(f, g), delta2, eps)
        }
    }
}

theorem rat_mul_uniform(r: Rat) {
    rat_uniform(compose(Real.from_rat, r.mul))
} by {
    let mul_r = compose(Real.from_rat, r.mul)
    forall(eps: Real) {
        if eps.is_positive {
            let reps1: Rat satisfy {
                reps1.is_positive and Real.from_rat(reps1) < eps
            }
            let reps2: Rat satisfy {
                reps2.is_positive and reps2 < reps1
            }
            Rat.0 <= r.abs
            let delta: Rat satisfy {
                delta.is_positive and r.abs * delta < reps2
            }
            forall(r1: Rat, r2: Rat) {
                if r1.is_close(r2, delta) {
                    (r * (r1 - r2)).abs <= r.abs * delta
                    r * r1 - r * r2 = r * (r1 - r2)
                    r.abs * delta < reps1
                    (r * r1 - r * r2).abs < reps1
                    Real.from_rat(r * r1).is_close(Real.from_rat(r * r2), eps)
                    mul_r(r1).is_close(mul_r(r2), eps)
                }
            }
            exists(k0: Rat, k1: Rat) {
                k0.is_close(k1, delta) and
                not mul_r(k0).is_close(mul_r(k1), eps)
            } or rat_condition(mul_r, delta, eps)
            rat_condition(mul_r, delta, eps)
            delta.is_positive and rat_condition(mul_r, delta, eps)
            exists(delta_inner: Rat) {
                delta_inner.is_positive and rat_condition(mul_r, delta_inner, eps)
            }
        }
    }
    rat_uniform(mul_r)
}

theorem close_add(a: Real, b: Real, c: Real, eps: Real) {
    a.is_close(b, eps)
    implies
    (a + c).is_close(b + c, eps)
} by {
    a + c - (b + c) = a + c - b - c
    a + c - b = a - b + c
    a - b + c - c = a - b
    (a + c - (b + c)).abs < eps = (a + c).is_close(b + c, eps)
    (a - b).abs < eps = a.is_close(b, eps)
}

theorem add_real_uniform(x: Real) {
    uniform(x.add)
} by {
    forall(eps: Real) {
        if eps.is_positive {
            forall(a: Real, b: Real) {
                if a.is_close(b, eps) {
                    x.add(a).is_close(x.add(b), eps)
                }
            }
            uniform_condition(x.add, eps, eps)
            eps.is_positive and uniform_condition(x.add, eps, eps)
            exists(delta: Real) {
                delta.is_positive and uniform_condition(x.add, delta, eps)
            }
        }
    }
}

theorem neg_uniform {
    uniform(Real.neg)
} by {
    forall(eps: Real) {
        if eps.is_positive {
            forall(a: Real, b: Real) {
                if a.is_close(b, eps) {
                    (-a).is_close(-b, eps)
                }
            }
            uniform_condition(Real.neg, eps, eps)
            eps.is_positive and uniform_condition(Real.neg, eps, eps)
            exists(delta: Real) {
                delta.is_positive and uniform_condition(Real.neg, delta, eps)
            }
        }
    }
}

/// Pointwise addition of two real-valued functions.
define add_fns[T](f: T -> Real, g: T -> Real, t: T) -> Real {
    f(t) + g(t)
}

theorem add_fns_uniform(f: Real -> Real, g: Real -> Real) {
    uniform(f) and uniform(g)
    implies
    uniform(add_fns(f, g))
} by {
    forall(eps: Real) {
        if eps.is_positive {
            let eps2: Real satisfy {
                eps2.is_positive and eps2 + eps2 < eps
            }
            let delta1: Real satisfy {
                delta1.is_positive and uniform_condition(f, delta1, eps2)
            }
            let delta2: Real satisfy {
                delta2.is_positive and uniform_condition(g, delta2, eps2)
            }
            let delta: Real satisfy {
                delta.is_positive and delta < delta1 and delta < delta2
            }

            forall(x: Real, y: Real) {
                if x.is_close(y, delta) {
                    (x - y).abs < delta = x.is_close(y, delta)
                    (x - y).abs < delta1 = x.is_close(y, delta1)
                    (x - y).abs < delta2 = x.is_close(y, delta2)
                    not (x - y).abs < delta or not delta < delta1 or (x - y).abs < delta1
                    not (x - y).abs < delta or not delta < delta2 or (x - y).abs < delta2
                    x.is_close(y, delta1)
                    x.is_close(y, delta2)
                    f(x).is_close(f(y), eps2)
                    g(x).is_close(g(y), eps2)
                    (f(x) + g(x)).is_close(f(y) + g(y), eps2 + eps2)
                    add_fns(f, g, x).is_close(add_fns(f, g, y), eps)
                }
            }
            exists(k0: Real, k1: Real) {
                k0.is_close(k1, delta) and
                not add_fns(f, g, k0).is_close(add_fns(f, g, k1), eps)
            } or uniform_condition(add_fns(f, g), delta, eps)
            uniform_condition(add_fns(f, g), delta, eps)
            delta.is_positive and uniform_condition(add_fns(f, g), delta, eps)
        }
    }
}

// Historically we proved some theorems around continuity and 2d continuity here.
// But this might not be the best order to prove things in.
// Same for the uniform continuity theorems.
/// True if f satisfies the delta-epsilon condition for continuity at (x,y).
define continuous2_condition(f: (Real, Real) -> Real, x: Real, y: Real, delta: Real, eps: Real) -> Bool {
    forall(x1: Real, y1: Real) {
        x1.is_close(x, delta) and y1.is_close(y, delta) implies
        f(x1, y1).is_close(f(x, y), eps)
    }
}

/// True if the two-variable function f is continuous at the point (x,y).
define continuous2_at(f: (Real, Real) -> Real, x: Real, y: Real) -> Bool {
    forall(eps: Real) {
        eps.is_positive implies exists(delta: Real) {
            delta.is_positive and
            continuous2_condition(f, x, y, delta, eps)
        }
    }
}

/// True if the two-variable function f is continuous everywhere.
define continuous2(f: (Real, Real) -> Real) -> Bool {
    forall(x: Real, y: Real) {
        continuous2_at(f, x, y)
    }
}

// Addition is continuous.
theorem add_continuous2 {
    continuous2(Real.add)
} by {
    forall(x: Real, y: Real) {
        forall(eps: Real) {
            if eps.is_positive {
                let delta: Real satisfy {
                    delta.is_positive and delta + delta < eps
                }
                forall(x1: Real, y1: Real) {
                    if x1.is_close(x, delta) and y1.is_close(y, delta) {
                        Real.add(x1, y1).is_close(Real.add(x, y), eps)
                    }
                }
                exists(k0: Real, k1: Real) {
                    k0.is_close(x, delta) and
                    k1.is_close(y, delta) and
                    not Real.add(k0, k1).is_close(Real.add(x, y), eps)
                } or continuous2_condition(Real.add, x, y, delta, eps)
                continuous2_condition(Real.add, x, y, delta, eps)
                delta.is_positive and continuous2_condition(Real.add, x, y, delta, eps)
            }
        }
        continuous2_at(Real.add, x, y)
    }
}

/// True if f satisfies the delta-epsilon condition for continuity at x.
define continuous_condition(f: Real -> Real, x: Real, delta: Real, eps: Real) -> Bool {
    forall(x1: Real) {
        x1.is_close(x, delta) implies
        f(x1).is_close(f(x), eps)
    }
}

/// True if f is continuous at the point x.
define continuous_at(f: Real -> Real, x: Real) -> Bool {
    forall(eps: Real) {
        eps.is_positive implies exists(delta: Real) {
            delta.is_positive and continuous_condition(f, x, delta, eps)
        }
    }
}

/// True if f is continuous everywhere on the reals.
define continuous(f: Real -> Real) -> Bool {
    forall(x: Real) {
        continuous_at(f, x)
    }
}

theorem uniform_imp_continuous(f: Real -> Real) {
    uniform(f)
    implies
    continuous(f)
} by {
        forall(x: Real) {
            forall(eps: Real) {
                if eps.is_positive {
                    let delta: Real satisfy {
                        delta.is_positive and uniform_condition(f, delta, eps)
                    }
                    forall(x1: Real) {
                        if x1.is_close(x, delta) {
                            f(x1).is_close(f(x), eps)
                        }
                    }
                    continuous_condition(f, x, delta, eps)
                    delta.is_positive and continuous_condition(f, x, delta, eps)
                    exists(delta_inner: Real) {
                        delta_inner.is_positive and continuous_condition(f, x, delta_inner, eps)
                    }
                }
        }
        continuous_at(f, x)
    }
    continuous(f)
}
