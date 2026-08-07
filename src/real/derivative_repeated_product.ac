from data.basic.function_algebra import pointwise_mul
from data.basic.functions import identity_fn
from real.derivative_basic import has_derivative_at, differentiable_at,
    identity_has_derivative_at
from real.derivative_product import derivative_pointwise_mul,
    derivative_pointwise_square, differentiable_pointwise_mul
from real.real_base import Real

/// A left-associated product of three differentiable functions satisfies the iterated product rule.
theorem derivative_pointwise_mul_three_left(
    f: Real -> Real, g: Real -> Real, h: Real -> Real,
    x0: Real, df: Real, dg: Real, dh: Real
) {
    has_derivative_at(f, x0, df) and has_derivative_at(g, x0, dg) and has_derivative_at(h, x0, dh)
    implies has_derivative_at(
        pointwise_mul(pointwise_mul(f, g), h),
        x0,
        pointwise_mul(f, g, x0) * dh + h(x0) * (f(x0) * dg + g(x0) * df)
    )
} by {
    derivative_pointwise_mul(f, g, x0, df, dg)
    has_derivative_at(pointwise_mul(f, g), x0, f(x0) * dg + g(x0) * df)
    derivative_pointwise_mul(pointwise_mul(f, g), h, x0, f(x0) * dg + g(x0) * df, dh)
    has_derivative_at(
        pointwise_mul(pointwise_mul(f, g), h),
        x0,
        pointwise_mul(f, g, x0) * dh + h(x0) * (f(x0) * dg + g(x0) * df)
    )
}

/// A right-associated product of three differentiable functions satisfies the iterated product rule.
theorem derivative_pointwise_mul_three_right(
    f: Real -> Real, g: Real -> Real, h: Real -> Real,
    x0: Real, df: Real, dg: Real, dh: Real
) {
    has_derivative_at(f, x0, df) and has_derivative_at(g, x0, dg) and has_derivative_at(h, x0, dh)
    implies has_derivative_at(
        pointwise_mul(f, pointwise_mul(g, h)),
        x0,
        f(x0) * (g(x0) * dh + h(x0) * dg) + pointwise_mul(g, h, x0) * df
    )
} by {
    derivative_pointwise_mul(g, h, x0, dg, dh)
    has_derivative_at(pointwise_mul(g, h), x0, g(x0) * dh + h(x0) * dg)
    derivative_pointwise_mul(f, pointwise_mul(g, h), x0, df, g(x0) * dh + h(x0) * dg)
    has_derivative_at(
        pointwise_mul(f, pointwise_mul(g, h)),
        x0,
        f(x0) * (g(x0) * dh + h(x0) * dg) + pointwise_mul(g, h, x0) * df
    )
}

/// Differentiability is preserved by left-associated triple products.
theorem differentiable_pointwise_mul_three_left(
    f: Real -> Real, g: Real -> Real, h: Real -> Real, x0: Real
) {
    differentiable_at(f, x0) and differentiable_at(g, x0) and differentiable_at(h, x0)
    implies differentiable_at(pointwise_mul(pointwise_mul(f, g), h), x0)
} by {
    differentiable_pointwise_mul(f, g, x0)
    differentiable_at(pointwise_mul(f, g), x0)
    differentiable_pointwise_mul(pointwise_mul(f, g), h, x0)
    differentiable_at(pointwise_mul(pointwise_mul(f, g), h), x0)
}

/// Differentiability is preserved by right-associated triple products.
theorem differentiable_pointwise_mul_three_right(
    f: Real -> Real, g: Real -> Real, h: Real -> Real, x0: Real
) {
    differentiable_at(f, x0) and differentiable_at(g, x0) and differentiable_at(h, x0)
    implies differentiable_at(pointwise_mul(f, pointwise_mul(g, h)), x0)
} by {
    differentiable_pointwise_mul(g, h, x0)
    differentiable_at(pointwise_mul(g, h), x0)
    differentiable_pointwise_mul(f, pointwise_mul(g, h), x0)
    differentiable_at(pointwise_mul(f, pointwise_mul(g, h)), x0)
}

/// Cubes of a differentiable function have the derivative obtained by the iterated product rule.
theorem derivative_pointwise_cube(f: Real -> Real, x0: Real, d: Real) {
    has_derivative_at(f, x0, d)
    implies has_derivative_at(
        pointwise_mul(pointwise_mul(f, f), f),
        x0,
        pointwise_mul(f, f, x0) * d + f(x0) * (f(x0) * d + f(x0) * d)
    )
} by {
    derivative_pointwise_mul_three_left(f, f, f, x0, d, d, d)
    has_derivative_at(
        pointwise_mul(pointwise_mul(f, f), f),
        x0,
        pointwise_mul(f, f, x0) * d + f(x0) * (f(x0) * d + f(x0) * d)
    )
}

/// Differentiability is preserved by cubing a function pointwise.
theorem differentiable_pointwise_cube(f: Real -> Real, x0: Real) {
    differentiable_at(f, x0) implies differentiable_at(pointwise_mul(pointwise_mul(f, f), f), x0)
} by {
    differentiable_pointwise_mul_three_left(f, f, f, x0)
    differentiable_at(pointwise_mul(pointwise_mul(f, f), f), x0)
}

/// The square of the identity function has the expected derivative expression.
theorem identity_square_has_derivative_at(x0: Real) {
    has_derivative_at(pointwise_mul(identity_fn[Real], identity_fn[Real]), x0, x0 * Real.1 + x0 * Real.1)
} by {
    identity_has_derivative_at(x0)
    derivative_pointwise_square(identity_fn[Real], x0, Real.1)
    identity_fn[Real](x0) = x0
    has_derivative_at(pointwise_mul(identity_fn[Real], identity_fn[Real]), x0, x0 * Real.1 + x0 * Real.1)
}

/// The square of the identity function is differentiable at every real point.
theorem identity_square_differentiable_at(x0: Real) {
    differentiable_at(pointwise_mul(identity_fn[Real], identity_fn[Real]), x0)
} by {
    identity_square_has_derivative_at(x0)
    exists(d: Real) {
        has_derivative_at(pointwise_mul(identity_fn[Real], identity_fn[Real]), x0, d)
    }
}

/// The cube of the identity function has the derivative expression from the triple product rule.
theorem identity_cube_has_derivative_at(x0: Real) {
    has_derivative_at(
        pointwise_mul(pointwise_mul(identity_fn[Real], identity_fn[Real]), identity_fn[Real]),
        x0,
        pointwise_mul(identity_fn[Real], identity_fn[Real], x0) * Real.1 +
        x0 * (x0 * Real.1 + x0 * Real.1)
    )
} by {
    identity_has_derivative_at(x0)
    derivative_pointwise_cube(identity_fn[Real], x0, Real.1)
    identity_fn[Real](x0) = x0
    has_derivative_at(
        pointwise_mul(pointwise_mul(identity_fn[Real], identity_fn[Real]), identity_fn[Real]),
        x0,
        pointwise_mul(identity_fn[Real], identity_fn[Real], x0) * Real.1 +
        x0 * (x0 * Real.1 + x0 * Real.1)
    )
}

/// The cube of the identity function is differentiable at every real point.
theorem identity_cube_differentiable_at(x0: Real) {
    differentiable_at(pointwise_mul(pointwise_mul(identity_fn[Real], identity_fn[Real]), identity_fn[Real]), x0)
} by {
    identity_cube_has_derivative_at(x0)
    exists(d: Real) {
        has_derivative_at(pointwise_mul(pointwise_mul(identity_fn[Real], identity_fn[Real]), identity_fn[Real]), x0, d)
    }
}
