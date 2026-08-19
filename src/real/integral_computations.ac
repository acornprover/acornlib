/// Concrete integral computations via the fundamental theorem of calculus.
///
/// This file computes the integrals of the first power functions and of the
/// trigonometric functions over their canonical intervals, and restates the
/// fundamental theorem of calculus (FTC part 2) that underlies them:
///   - the integral of x over [0, 1] is 1/2 (antiderivative x^2 / 2),
///   - the integral of x^2 over [0, 1] is 1/3 (antiderivative x^3 / 3),
///   - the integral of Real.sin over [0, pi] is 2 (antiderivative -Real.cos),
///   - the integral of Real.cos over [0, pi/2] is 1 (antiderivative Real.sin),
///   - the FTC: if F' = f and f is integrable and bounded on [a, b], then
///     integral(f, a, b) = F(b) - F(a) (restated from integral_exp.ac).
///
/// The route follows integral_exp.ac and integral_trig.ac: integrability of
/// the power functions is established by comparing upper and lower Darboux
/// sums over the uniform partitions of [a, b], whose difference is at most
/// (L * (b - a) * (b - a)) / (n + 1) for an L-Lipschitz integrand (the
/// generalization fn_integrable_gen_sym of the one-Lipschitz fn_integrable of
/// integral_trig.ac), and the value is then forced by the mean-value-theorem
/// sandwich of the Darboux sums between the increments of the antiderivative
/// (ftc2_general).

from nat import Nat, from_nat, from_nat_add, lt_imp_lte_suc, pow_one
from rat import Rat
from order import lte_antisymm, lte_trans, lt_trans, lt_imp_lte, lt_imp_ne, lt_imp_ne_symm, not_lt_imp_gte, lt_of_lt_of_lte, lt_of_lte_of_lt, not_lte_imp_gt, not_lt_self
from real.real_field import Real, mul_div, mul_inverse, div_mul_cancel_left
from real.real_base import add_comm, add_assoc, neg_distrib, abs_neg, neg_neg, lte_abs, abs_gte_zero, lte_lt_trans, lte_add_right, gt_zero_imp_pos, pos_gt_zero, sub_cancels, sub_moves_sides, lte_self, neg_lt_zero, neg_zero
from real.real_ring import from_nat_is_from_rat, mul_pos_pos, lt_mul_pos_left, lte_mul_nonneg_right, mul_sub_distrib_left, mul_abs, square_nonneg, non_neg_imp_zero_lte, real_mul_comm, mul_assoc, mul_distrib_right, mul_distrib_left, mul_one_left, mul_one_right, mul_zero_left
from real.real_seq import sub_zero_imp_eq
from real.real_series import pow_nonneg, abs_pow, const_seq
from real.harmonic import from_nat_suc_pos_real, rat_from_nat_lte_of_nat_lte
from real.exp import pow_suc, two, two_positive, div_two_is_mul_half, one_div_two_eq_half
from real.cbrt import three, one_third, three_positive
from real.sqrt_inequalities import one_half_mul_double
from real.trig import sin_zero, cos_zero
from real.pi import pi_over_two, pi, sin_pi_over_two_one, cos_pi_neg_one, sin_continuous, cos_continuous, pi_pos, pi_over_two_pos
from real.derivative_trig import sin_is_derivative_fn, abs_of_nonneg, sq_lte_base
from real.derivative_basic import has_derivative_at, has_derivative_at_unique, sub_ne_zero_of_ne, identity_has_derivative_at, constant_has_derivative_at
from real.derivative_rules import derivative_pointwise_add, derivative_pointwise_neg
from real.derivative_chain import derivative_compose
from real.derivative_continuity import div_mul_cancel_denominator
from real.mean_value import mean_value_theorem, secant_slope, lte_imp_neg_lte_neg
from real.calculus_api import is_derivative_fn, is_derivative_fn_at, is_derivative_fn_iff
from real.calculus_api_standard_examples import derivative_fn_square_real
from real.calculus_quotient_api import derivative_fn_div_const
from real.calculus_quotient_continuity_examples import is_derivative_fn_imp_continuous
from real.continuity_base import continuous, continuous_at
from real.continuity_square import square_real, continuous_square_real, square_real_eq_pointwise_mul_identity
from real.continuity_cube import cube_real, continuous_cube_real, cube_real_eq_pointwise_mul_square_identity
from real.derivative_polynomial_chain import cube_real_has_derivative_at, square_real_has_derivative_at
from real.derivative_quotient import pointwise_div_real
from data.basic.function_algebra import pointwise_add, pointwise_neg, pointwise_mul, pointwise_add_apply
from data.basic.functions import identity_fn, compose, function_eq_transport_predicate_rev
from real.supremum import completeness, is_nonempty, has_upper_bound, is_set_supremum, is_set_upper_bound, is_set_infimum, is_set_lower_bound, set_member_le_supremum, set_supremum_le_upper_bound, set_upper_bound_contains_le, function_image_contains, negate_set, negate_set_contains
from real.integral import integral, is_integrable, interval_contains, interval_set, interval_image, interval_inf, interval_sup, interval_inf_spec, interval_sup_spec, lower_sum, upper_sum, lower_sum_set, upper_sum_set, lower_sum_contains, upper_sum_contains, partition_step_lower, partition_step_upper, is_partition, partition_start, partition_end, partition_mono, partition_point_in_interval, partition_monotone, diff_step, telescope, partial_lte, image_lower_bound, interval_set_contains_left, interval_set_contains_right, interval_contains_left, interval_contains_right, interval_contains_mono, sub_nonneg, integral_spec, set_supremum_unique, set_infimum_unique, sup_le_of_upper_bound, trivial_partition, trivial_partition_is_partition, neg_lte_flip, set_infimum_is_lower_bound, set_lower_bound_contains_le, set_lower_bound_le_infimum, has_lower_bound
from real.integral import neg_upper_bound_of_lower, negate_set_nonempty, inf_of_neg_sup
from real.integral_trig import fn_integrable, lipschitz_uniform_upper_minus_lower, interval_sup_sub_inf_le_lipschitz, interval_abs_diff_le_width, fn_lower_sum_set_sup_exists, fn_upper_sum_set_inf_exists, fn_lower_sum_set_bounded_above, fn_upper_sum_set_bounded_below, integral_sin_zero_pi, integral_cos_zero_pi_over_two, uniform_mesh
from real.integral_exp import ftc2_general, lower_sum_le_g_diff, g_diff_le_upper_sum, image_upper_bound, uniform_partition, uniform_partition_start, uniform_partition_end, uniform_partition_monotone, uniform_partition_is_partition, uniform_partition_width, div_nonneg_pos_denom, from_nat_lte_mono, mul_frac_right, nonneg_frac_all_n_imp_zero, add_sub_cancel_shift, sub_add_cancel_shift, add_one_mul_sub
from real.double_sum import partial_sub_seq, sub_seq
from list import partial, partial_pointwise_eq, partial_scalar_mul
from algebra.semigroup import mul_fn
from algebra.add_ordered_group import add_le_add, add_le_add_right
from ordered_field import mul_le_mul_of_nonneg_right
from data.basic.set import Set, maps_into_set_image
from real.am_gm import partial_const

numerals Real
numerals Nat

// ---------------------------------------------------------------------------
// Antiderivatives of the power functions
// ---------------------------------------------------------------------------

/// The antiderivative x^2 / 2 of the identity function has derivative the
/// identity function.
theorem half_square_is_derivative_fn {
    is_derivative_fn(pointwise_div_real(square_real, constant[Real, Real](two)), identity_fn[Real])
} by {
    forall(x: Real) {
        derivative_fn_square_real
        is_derivative_fn(square_real, pointwise_add(identity_fn[Real], identity_fn[Real]))
        derivative_fn_div_const(square_real, pointwise_add(identity_fn[Real], identity_fn[Real]), two)
        is_derivative_fn(pointwise_div_real(square_real, constant[Real, Real](two)),
            pointwise_div_real(pointwise_add(identity_fn[Real], identity_fn[Real]), constant[Real, Real](two)))
        is_derivative_fn_at(pointwise_div_real(square_real, constant[Real, Real](two)),
            pointwise_div_real(pointwise_add(identity_fn[Real], identity_fn[Real]), constant[Real, Real](two)), x)
        has_derivative_at(pointwise_div_real(square_real, constant[Real, Real](two)), x,
            pointwise_div_real(pointwise_add(identity_fn[Real], identity_fn[Real]), constant[Real, Real](two), x))
        identity_fn[Real](x) = x
        pointwise_add_apply(identity_fn[Real], identity_fn[Real], x)
        pointwise_add(identity_fn[Real], identity_fn[Real], x) = identity_fn[Real](x) + identity_fn[Real](x)
        identity_fn[Real](x) + identity_fn[Real](x) = x + x
        pointwise_add(identity_fn[Real], identity_fn[Real], x) = x + x
        constant[Real, Real](two, x) = two
        pointwise_div_real(pointwise_add(identity_fn[Real], identity_fn[Real]), constant[Real, Real](two), x) =
            (x + x) / two
        div_two_is_mul_half(x + x)
        (x + x) / two = (x + x) * Real.one_half
        real_mul_comm(x + x, Real.one_half)
        (x + x) * Real.one_half = Real.one_half * (x + x)
        one_half_mul_double(x)
        Real.one_half * (x + x) = x
        (x + x) / two = x
        pointwise_div_real(pointwise_add(identity_fn[Real], identity_fn[Real]), constant[Real, Real](two), x) = x
        identity_fn[Real](x) = x
        pointwise_div_real(pointwise_add(identity_fn[Real], identity_fn[Real]), constant[Real, Real](two), x) =
            identity_fn[Real](x)
        has_derivative_at(pointwise_div_real(square_real, constant[Real, Real](two)), x, identity_fn[Real](x))
    }
}

/// The antiderivative x^3 / 3 of the square function has derivative the
/// square function.
theorem third_cube_is_derivative_fn {
    is_derivative_fn(pointwise_div_real(cube_real, constant[Real, Real](three)), square_real)
} by {
    forall(x: Real) {
        cube_real_has_derivative_at(x)
        has_derivative_at(cube_real, x,
            square_real(x) * Real.1 + x * (x * Real.1 + x * Real.1))
        // The derivative value at x is three times square_real(x).
        pointwise_mul(constant[Real, Real](three), square_real, x) =
            constant[Real, Real](three, x) * square_real(x)
        constant[Real, Real](three, x) = three
        pointwise_mul(constant[Real, Real](three), square_real, x) = three * square_real(x)
        // Normalize the raw derivative value to three * square_real(x).
        square_real(x) * Real.1 + x * (x * Real.1 + x * Real.1) = x * x + x * (x + x)
        x * x + x * (x + x) = x * x + x * x + x * x
        three = Real.1 + Real.1 + Real.1
        three * square_real(x) = (Real.1 + Real.1 + Real.1) * (x * x)
        mul_distrib_right(Real.1 + Real.1, Real.1, x * x)
        (Real.1 + Real.1 + Real.1) * (x * x) =
            (Real.1 + Real.1) * (x * x) + Real.1 * (x * x)
        mul_distrib_right(Real.1, Real.1, x * x)
        (Real.1 + Real.1) * (x * x) = Real.1 * (x * x) + Real.1 * (x * x)
        mul_one_left(x * x)
        Real.1 * (x * x) = x * x
        (Real.1 + Real.1) * (x * x) + Real.1 * (x * x) = x * x + x * x + x * x
        (Real.1 + Real.1 + Real.1) * (x * x) = x * x + x * x + x * x
        x * x + x * x + x * x = three * square_real(x)
        square_real(x) * Real.1 + x * (x * Real.1 + x * Real.1) = three * square_real(x)
        has_derivative_at(cube_real, x, pointwise_mul(constant[Real, Real](three), square_real, x))
    }
    // cube has global derivative three * square.
    is_derivative_fn(cube_real, pointwise_mul(constant[Real, Real](three), square_real))
    derivative_fn_div_const(cube_real, pointwise_mul(constant[Real, Real](three), square_real), three)
    is_derivative_fn(pointwise_div_real(cube_real, constant[Real, Real](three)),
        pointwise_div_real(pointwise_mul(constant[Real, Real](three), square_real), constant[Real, Real](three)))
    forall(x: Real) {
        is_derivative_fn_at(pointwise_div_real(cube_real, constant[Real, Real](three)),
            pointwise_div_real(pointwise_mul(constant[Real, Real](three), square_real), constant[Real, Real](three)), x)
        has_derivative_at(pointwise_div_real(cube_real, constant[Real, Real](three)), x,
            pointwise_div_real(pointwise_mul(constant[Real, Real](three), square_real), constant[Real, Real](three), x))
        pointwise_div_real(pointwise_mul(constant[Real, Real](three), square_real), constant[Real, Real](three), x) =
            pointwise_mul(constant[Real, Real](three), square_real, x) / constant[Real, Real](three, x)
        pointwise_div_real(pointwise_mul(constant[Real, Real](three), square_real), constant[Real, Real](three), x) =
            (three * square_real(x)) / three
        three_positive
        three > Real.0
        three != Real.0
        // (three * square_real(x)) / three = square_real(x)
        div_mul_cancel_left(three, square_real(x))
        (three * square_real(x)) / three = square_real(x)
        pointwise_div_real(pointwise_mul(constant[Real, Real](three), square_real), constant[Real, Real](three), x) =
            square_real(x)
        has_derivative_at(pointwise_div_real(cube_real, constant[Real, Real](three)), x, square_real(x))
    }
}

// ---------------------------------------------------------------------------
// Bounds of the integrands on the unit interval
// ---------------------------------------------------------------------------

/// The identity function is one-Lipschitz.
theorem identity_lipschitz_on_unit {
    Real.0 <= Real.1 implies forall(u: Real, v: Real) {
        (identity_fn[Real](u) - identity_fn[Real](v)).abs <= (u - v).abs
    }
} by {
    if Real.0 <= Real.1 {
        forall(u: Real, v: Real) {
            identity_fn[Real](u) = u
            identity_fn[Real](v) = v
            identity_fn[Real](u) - identity_fn[Real](v) = u - v
            (identity_fn[Real](u) - identity_fn[Real](v)).abs = (u - v).abs
            lte_self((u - v).abs)
            (u - v).abs <= (u - v).abs
            (identity_fn[Real](u) - identity_fn[Real](v)).abs <= (u - v).abs
        }
    }
}

/// The identity function is bounded below by -1 on [0, 1].
theorem identity_lower_bound_on_unit {
    Real.0 <= Real.1 implies forall(t: Real) {
        interval_contains(Real.0, Real.1, t) implies -Real.1 <= identity_fn[Real](t)
    }
} by {
    if Real.0 <= Real.1 {
        forall(t: Real) {
            if interval_contains(Real.0, Real.1, t) {
                interval_contains_left(Real.0, Real.1, t)
                Real.0 <= t
                interval_contains_right(Real.0, Real.1, t)
                t <= Real.1
                identity_fn[Real](t) = t
                Real.0 <= identity_fn[Real](t)
                lte_imp_neg_lte_neg(Real.0, Real.1)
                -Real.1 <= -Real.0
                neg_zero
                -Real.0 = Real.0
                -Real.1 <= Real.0
                lte_trans[Real](-Real.1, Real.0, identity_fn[Real](t))
                -Real.1 <= identity_fn[Real](t)
            }
        }
    }
}

/// The identity function is bounded above by 1 on [0, 1].
theorem identity_upper_bound_on_unit {
    Real.0 <= Real.1 implies forall(t: Real) {
        interval_contains(Real.0, Real.1, t) implies identity_fn[Real](t) <= Real.1
    }
} by {
    if Real.0 <= Real.1 {
        forall(t: Real) {
            if interval_contains(Real.0, Real.1, t) {
                interval_contains_left(Real.0, Real.1, t)
                Real.0 <= t
                interval_contains_right(Real.0, Real.1, t)
                t <= Real.1
                identity_fn[Real](t) = t
                identity_fn[Real](t) <= Real.1
            }
        }
    }
}

// ---------------------------------------------------------------------------
// The integral of x over [0, 1]
// ---------------------------------------------------------------------------

/// The integral of the identity function over [0, 1] is 1/2.
///
/// The antiderivative is x^2 / 2; the identity is one-Lipschitz and bounded
/// by one in absolute value on [0, 1], so fn_integrable applies and
/// ftc2_general gives integral(x, 0, 1) = 1/2 - 0.
theorem integral_identity_unit {
    integral(identity_fn[Real], Real.0, Real.1) = Real.one_half
} by {
    Real.1 > Real.0
    lt_imp_lte(Real.0, Real.1)
    Real.0 <= Real.1
    if Real.0 <= Real.1 {
        // The identity is one-Lipschitz and bounded by one in absolute value on [0, 1].
        identity_lipschitz_on_unit
        identity_lower_bound_on_unit
        identity_upper_bound_on_unit
        // The antiderivative x^2 / 2 is continuous with derivative the identity.
        half_square_is_derivative_fn
        is_derivative_fn(pointwise_div_real(square_real, constant[Real, Real](two)), identity_fn[Real])
        is_derivative_fn_imp_continuous(pointwise_div_real(square_real, constant[Real, Real](two)), identity_fn[Real])
        continuous(pointwise_div_real(square_real, constant[Real, Real](two)))
        // Re-expose the hypotheses as assumptions so that fn_integrable and
        // ftc2_general can be applied.
        if continuous(pointwise_div_real(square_real, constant[Real, Real](two))) {
            if is_derivative_fn(pointwise_div_real(square_real, constant[Real, Real](two)), identity_fn[Real]) {
                if forall(u: Real, v: Real) {
                    (identity_fn[Real](u) - identity_fn[Real](v)).abs <= (u - v).abs
                } {
                    if forall(t: Real) {
                        interval_contains(Real.0, Real.1, t) implies -Real.1 <= identity_fn[Real](t)
                    } {
                        if forall(t: Real) {
                            interval_contains(Real.0, Real.1, t) implies identity_fn[Real](t) <= Real.1
                        } {
                            // Integrability of the identity on [0, 1].
                            fn_integrable(identity_fn[Real],
                                pointwise_div_real(square_real, constant[Real, Real](two)), Real.0, Real.1)
                            is_integrable(identity_fn[Real], Real.0, Real.1)
                            // The fundamental theorem of calculus forces the value.
                            ftc2_general(identity_fn[Real],
                                pointwise_div_real(square_real, constant[Real, Real](two)),
                                Real.0, Real.1, -Real.1, Real.1)
                            integral(identity_fn[Real], Real.0, Real.1) =
                                pointwise_div_real(square_real, constant[Real, Real](two), Real.1) -
                                pointwise_div_real(square_real, constant[Real, Real](two), Real.0)
                            // The antiderivative at 1 is 1/2.
                            pointwise_div_real(square_real, constant[Real, Real](two), Real.1) =
                                square_real(Real.1) / constant[Real, Real](two, Real.1)
                            constant[Real, Real](two, Real.1) = two
                            square_real(Real.1) = Real.1
                            pointwise_div_real(square_real, constant[Real, Real](two), Real.1) = Real.1 / two
                            one_div_two_eq_half
                            Real.1 / two = Real.one_half
                            pointwise_div_real(square_real, constant[Real, Real](two), Real.1) = Real.one_half
                            // The antiderivative at 0 is 0.
                            pointwise_div_real(square_real, constant[Real, Real](two), Real.0) =
                                square_real(Real.0) / constant[Real, Real](two, Real.0)
                            constant[Real, Real](two, Real.0) = two
                            square_real(Real.0) = Real.0
                            pointwise_div_real(square_real, constant[Real, Real](two), Real.0) = Real.0 / two
                            Real.0 / two = Real.0
                            pointwise_div_real(square_real, constant[Real, Real](two), Real.0) = Real.0
                            // 1/2 - 0 = 1/2.
                            Real.one_half - Real.0 = Real.one_half
                            integral(identity_fn[Real], Real.0, Real.1) = Real.one_half
                        }
                    }
                }
            }
        }
    }
}

// ---------------------------------------------------------------------------
// The integrals of sine and cosine over their canonical intervals
// ---------------------------------------------------------------------------

/// The integral of sine over [0, pi] is two.
///
/// Restated from integral_trig.ac, where it is proved by the fundamental
/// theorem of calculus with antiderivative -Real.cos.
theorem integral_sin_zero_pi_value {
    integral(Real.sin, Real.0, pi) = two
} by {
    integral_sin_zero_pi
    integral(Real.sin, Real.0, pi) = two
}

/// The integral of cosine over [0, pi/2] is one.
///
/// Restated from integral_trig.ac, where it is proved by the fundamental
/// theorem of calculus with antiderivative Real.sin.
theorem integral_cos_zero_pi_over_two_value {
    integral(Real.cos, Real.0, pi_over_two) = Real.1
} by {
    integral_cos_zero_pi_over_two
    integral(Real.cos, Real.0, pi_over_two) = Real.1
}

// ---------------------------------------------------------------------------
// The fundamental theorem of calculus (restatement)
// ---------------------------------------------------------------------------

/// The fundamental theorem of calculus, part 2: if g is continuous with
/// pointwise derivative f, and f is integrable and bounded on [a, b], then
/// the integral of f over [a, b] is g(b) - g(a).
///
/// Restated from ftc2_general of integral_exp.ac; the proof there applies the
/// mean value theorem on each subinterval of a partition and sandwiches the
/// Darboux sums between the increments of g.
theorem ftc2_restated(f: Real -> Real, g: Real -> Real, a: Real, b: Real, lb: Real, ub: Real) {
    a <= b and continuous(g) and is_derivative_fn(g, f) and is_integrable(f, a, b) and
    (forall(t: Real) { interval_contains(a, b, t) implies lb <= f(t) }) and
    (forall(t: Real) { interval_contains(a, b, t) implies f(t) <= ub })
    implies integral(f, a, b) = g(b) - g(a)
} by {
    if a <= b and continuous(g) and is_derivative_fn(g, f) and is_integrable(f, a, b) and
       (forall(t: Real) { interval_contains(a, b, t) implies lb <= f(t) }) and
       (forall(t: Real) { interval_contains(a, b, t) implies f(t) <= ub }) {
        ftc2_general(f, g, a, b, lb, ub)
        integral(f, a, b) = g(b) - g(a)
    }
}

// ---------------------------------------------------------------------------
// Generalized Lipschitz integrability with an explicit Lipschitz constant
// ---------------------------------------------------------------------------
//
// The integrability machinery of integral_trig.ac (fn_integrable,
// lipschitz_uniform_upper_minus_lower and interval_sup_sub_inf_le_lipschitz)
// is specialized to one-Lipschitz functions bounded by one in absolute value.
// The theorems below generalize it to c-Lipschitz functions bounded by m in
// absolute value on [a, b], which is what the square function needs on [0, 1]
// (it is two-Lipschitz there).

/// If f is c-Lipschitz on [x, y] and bounded there, its supremum and infimum
/// over [x, y] differ by at most c * (y - x).
theorem interval_sup_sub_inf_le_lipschitz_gen(f: Real -> Real, c: Real, x: Real, y: Real) {
    x <= y and is_nonempty(interval_set(x, y)) and has_upper_bound(interval_image(f, x, y)) and
    has_lower_bound(interval_image(f, x, y)) and
    (forall(u: Real, v: Real) {
        interval_contains(x, y, u) and interval_contains(x, y, v) implies (f(u) - f(v)).abs <= c * (y - x)
    })
    implies interval_sup(f, x, y) - interval_inf(f, x, y) <= c * (y - x)
} by {
    if x <= y and is_nonempty(interval_set(x, y)) and has_upper_bound(interval_image(f, x, y)) and
       has_lower_bound(interval_image(f, x, y)) and
       (forall(u: Real, v: Real) {
           interval_contains(x, y, u) and interval_contains(x, y, v) implies (f(u) - f(v)).abs <= c * (y - x)
       }) {
        interval_sup_spec(f, x, y)
        is_set_supremum(interval_image(f, x, y), interval_sup(f, x, y))
        forall(v: Real) {
            if interval_image(f, x, y).contains(v) {
                interval_image(f, x, y).contains(v) = function_image_contains(f, interval_set(x, y), v)
                function_image_contains(f, interval_set(x, y), v)
                let t: Real satisfy {
                    interval_set(x, y).contains(t) and v = f(t)
                }
                interval_set(x, y).contains(t) = interval_contains(x, y, t)
                interval_contains(x, y, t)
                forall(w: Real) {
                    if interval_image(f, x, y).contains(w) {
                        interval_image(f, x, y).contains(w) = function_image_contains(f, interval_set(x, y), w)
                        function_image_contains(f, interval_set(x, y), w)
                        let s: Real satisfy {
                            interval_set(x, y).contains(s) and w = f(s)
                        }
                        interval_set(x, y).contains(s) = interval_contains(x, y, s)
                        interval_contains(x, y, s)
                        forall(u: Real, v0: Real) {
                            interval_contains(x, y, u) and interval_contains(x, y, v0) implies (f(u) - f(v0)).abs <= c * (y - x)
                        }
                        interval_contains(x, y, s) and interval_contains(x, y, t) implies (f(s) - f(t)).abs <= c * (y - x)
                        interval_contains(x, y, s) and interval_contains(x, y, t)
                        (f(s) - f(t)).abs <= c * (y - x)
                        lte_abs(f(s) - f(t))
                        f(s) - f(t) <= (f(s) - f(t)).abs
                        lte_trans[Real](f(s) - f(t), (f(s) - f(t)).abs, c * (y - x))
                        f(s) - f(t) <= c * (y - x)
                        add_le_add_right[Real](f(s) - f(t), c * (y - x), f(t))
                        f(s) - f(t) + f(t) <= c * (y - x) + f(t)
                        f(s) - f(t) + f(t) = f(s)
                        f(s) <= c * (y - x) + f(t)
                        c * (y - x) + f(t) = f(t) + c * (y - x)
                        f(s) <= f(t) + c * (y - x)
                        w = f(s)
                        v = f(t)
                        w <= v + c * (y - x)
                    }
                }
                is_set_upper_bound(interval_image(f, x, y), v + c * (y - x))
                sup_le_of_upper_bound(interval_image(f, x, y), interval_sup(f, x, y), v + c * (y - x))
                interval_sup(f, x, y) <= v + c * (y - x)
                add_le_add_right[Real](interval_sup(f, x, y), v + c * (y - x), -(c * (y - x)))
                interval_sup(f, x, y) + -(c * (y - x)) <= v + c * (y - x) + -(c * (y - x))
                interval_sup(f, x, y) - c * (y - x) = interval_sup(f, x, y) + -(c * (y - x))
                v + c * (y - x) + -(c * (y - x)) = v + c * (y - x) - c * (y - x)
                interval_sup(f, x, y) - c * (y - x) <= v + c * (y - x) - c * (y - x)
                v + c * (y - x) - c * (y - x) = v
                interval_sup(f, x, y) - c * (y - x) <= v
            }
        }
        is_set_lower_bound(interval_image(f, x, y), interval_sup(f, x, y) - c * (y - x))
        interval_inf_spec(f, x, y)
        is_set_infimum(interval_image(f, x, y), interval_inf(f, x, y))
        set_lower_bound_le_infimum(interval_image(f, x, y), interval_inf(f, x, y),
            interval_sup(f, x, y) - c * (y - x))
        interval_sup(f, x, y) - c * (y - x) <= interval_inf(f, x, y)
        add_le_add_right[Real](interval_sup(f, x, y) - c * (y - x), interval_inf(f, x, y), c * (y - x))
        interval_sup(f, x, y) - c * (y - x) + c * (y - x) <= interval_inf(f, x, y) + c * (y - x)
        interval_sup(f, x, y) - c * (y - x) + c * (y - x) = interval_sup(f, x, y)
        interval_sup(f, x, y) <= interval_inf(f, x, y) + c * (y - x)
        add_le_add_right[Real](interval_sup(f, x, y), interval_inf(f, x, y) + c * (y - x),
            -interval_inf(f, x, y))
        interval_sup(f, x, y) + -interval_inf(f, x, y) <= interval_inf(f, x, y) + c * (y - x) + -interval_inf(f, x, y)
        interval_sup(f, x, y) - interval_inf(f, x, y) = interval_sup(f, x, y) + -interval_inf(f, x, y)
        interval_inf(f, x, y) + c * (y - x) + -interval_inf(f, x, y) =
            interval_inf(f, x, y) + c * (y - x) - interval_inf(f, x, y)
        add_comm(interval_inf(f, x, y), c * (y - x))
        interval_inf(f, x, y) + c * (y - x) = c * (y - x) + interval_inf(f, x, y)
        interval_inf(f, x, y) + c * (y - x) - interval_inf(f, x, y) =
            c * (y - x) + interval_inf(f, x, y) - interval_inf(f, x, y)
        sub_cancels(c * (y - x), interval_inf(f, x, y))
        c * (y - x) + interval_inf(f, x, y) - interval_inf(f, x, y) = c * (y - x)
        interval_inf(f, x, y) + c * (y - x) - interval_inf(f, x, y) = c * (y - x)
        interval_sup(f, x, y) - interval_inf(f, x, y) <= c * (y - x)
    }
}

/// If f is c-Lipschitz and bounded by m in absolute value on [a, b], the
/// difference of its upper and lower Darboux sums over the uniform partition
/// of [a, b] into n + 1 equal parts is at most (c * (b - a) * (b - a)) / (n + 1).
theorem lipschitz_uniform_upper_minus_lower_gen(f: Real -> Real, c: Real, m: Real, a: Real, b: Real, n: Nat) {
    a <= b and Real.0 <= c and
    (forall(u: Real, v: Real) {
        interval_contains(a, b, u) and interval_contains(a, b, v) implies (f(u) - f(v)).abs <= c * (u - v).abs
    }) and
    (forall(t: Real) { interval_contains(a, b, t) implies -m <= f(t) }) and
    (forall(t: Real) { interval_contains(a, b, t) implies f(t) <= m })
    implies upper_sum(f, uniform_partition(a, b, n), n.suc) -
        lower_sum(f, uniform_partition(a, b, n), n.suc) <= (c * (b - a) * (b - a)) / from_nat[Real](n.suc)
} by {
    if a <= b and Real.0 <= c and
       (forall(u: Real, v: Real) {
           interval_contains(a, b, u) and interval_contains(a, b, v) implies (f(u) - f(v)).abs <= c * (u - v).abs
       }) and
       (forall(t: Real) { interval_contains(a, b, t) implies -m <= f(t) }) and
       (forall(t: Real) { interval_contains(a, b, t) implies f(t) <= m }) {
        uniform_partition_is_partition(a, b, n)
        is_partition(uniform_partition(a, b, n), a, b, n.suc)
        uniform_mesh(a, b, n) = (b - a) / from_nat[Real](n.suc)
        forall(i: Nat) {
            if i < n.suc {
                uniform_partition_width(a, b, n, i)
                uniform_partition(a, b, n, i + 1) - uniform_partition(a, b, n, i) =
                    (b - a) / from_nat[Real](n.suc)
                uniform_partition(a, b, n, i + 1) - uniform_partition(a, b, n, i) =
                    uniform_mesh(a, b, n)
                lt_imp_lte_suc(i, n.suc)
                i + 1 <= n.suc
                i <= i + 1
                lt_imp_lte(i, n.suc)
                i <= n.suc
                partition_mono(uniform_partition(a, b, n), a, b, n.suc, i, i + 1)
                uniform_partition(a, b, n, i) <= uniform_partition(a, b, n, i + 1)
                interval_set_contains_left(uniform_partition(a, b, n, i), uniform_partition(a, b, n, i + 1))
                interval_set(uniform_partition(a, b, n, i), uniform_partition(a, b, n, i + 1)).contains(
                    uniform_partition(a, b, n, i))
                exists(z: Real) {
                    interval_set(uniform_partition(a, b, n, i), uniform_partition(a, b, n, i + 1)).contains(z)
                }
                is_nonempty(interval_set(uniform_partition(a, b, n, i), uniform_partition(a, b, n, i + 1)))
                // f is bounded above by m on the subinterval.
                forall(t: Real) {
                    if interval_contains(uniform_partition(a, b, n, i), uniform_partition(a, b, n, i + 1), t) {
                        interval_contains_left(uniform_partition(a, b, n, i), uniform_partition(a, b, n, i + 1), t)
                        uniform_partition(a, b, n, i) <= t
                        interval_contains_right(uniform_partition(a, b, n, i), uniform_partition(a, b, n, i + 1), t)
                        t <= uniform_partition(a, b, n, i + 1)
                        partition_point_in_interval(uniform_partition(a, b, n), a, b, n.suc, i)
                        interval_contains(a, b, uniform_partition(a, b, n, i))
                        partition_point_in_interval(uniform_partition(a, b, n), a, b, n.suc, i + 1)
                        interval_contains(a, b, uniform_partition(a, b, n, i + 1))
                        interval_contains_mono(a, b, uniform_partition(a, b, n, i),
                            uniform_partition(a, b, n, i + 1), t)
                        interval_contains(a, b, t)
                        forall(t0: Real) {
                            interval_contains(a, b, t0) implies f(t0) <= m
                        }
                        interval_contains(a, b, t) implies f(t) <= m
                        f(t) <= m
                    }
                }
                image_upper_bound(f, uniform_partition(a, b, n, i), uniform_partition(a, b, n, i + 1), m)
                is_set_upper_bound(interval_image(f, uniform_partition(a, b, n, i),
                    uniform_partition(a, b, n, i + 1)), m)
                exists(b0: Real) {
                    is_set_upper_bound(interval_image(f, uniform_partition(a, b, n, i),
                        uniform_partition(a, b, n, i + 1)), b0)
                }
                has_upper_bound(interval_image(f, uniform_partition(a, b, n, i),
                    uniform_partition(a, b, n, i + 1)))
                // f is bounded below by -m on the subinterval.
                forall(t: Real) {
                    if interval_contains(uniform_partition(a, b, n, i), uniform_partition(a, b, n, i + 1), t) {
                        interval_contains_left(uniform_partition(a, b, n, i), uniform_partition(a, b, n, i + 1), t)
                        uniform_partition(a, b, n, i) <= t
                        interval_contains_right(uniform_partition(a, b, n, i), uniform_partition(a, b, n, i + 1), t)
                        t <= uniform_partition(a, b, n, i + 1)
                        partition_point_in_interval(uniform_partition(a, b, n), a, b, n.suc, i)
                        interval_contains(a, b, uniform_partition(a, b, n, i))
                        partition_point_in_interval(uniform_partition(a, b, n), a, b, n.suc, i + 1)
                        interval_contains(a, b, uniform_partition(a, b, n, i + 1))
                        interval_contains_mono(a, b, uniform_partition(a, b, n, i),
                            uniform_partition(a, b, n, i + 1), t)
                        interval_contains(a, b, t)
                        forall(t0: Real) {
                            interval_contains(a, b, t0) implies -m <= f(t0)
                        }
                        interval_contains(a, b, t) implies -m <= f(t)
                        -m <= f(t)
                    }
                }
                image_lower_bound(f, uniform_partition(a, b, n, i), uniform_partition(a, b, n, i + 1), -m)
                is_set_lower_bound(interval_image(f, uniform_partition(a, b, n, i),
                    uniform_partition(a, b, n, i + 1)), -m)
                exists(b0: Real) {
                    is_set_lower_bound(interval_image(f, uniform_partition(a, b, n, i),
                        uniform_partition(a, b, n, i + 1)), b0)
                }
                has_lower_bound(interval_image(f, uniform_partition(a, b, n, i),
                    uniform_partition(a, b, n, i + 1)))
                // f is c-Lipschitz on the subinterval with respect to the mesh.
                forall(u: Real, v: Real) {
                    if interval_contains(uniform_partition(a, b, n, i), uniform_partition(a, b, n, i + 1), u) and
                       interval_contains(uniform_partition(a, b, n, i), uniform_partition(a, b, n, i + 1), v) {
                        interval_contains_left(uniform_partition(a, b, n, i), uniform_partition(a, b, n, i + 1), u)
                        uniform_partition(a, b, n, i) <= u
                        interval_contains_right(uniform_partition(a, b, n, i), uniform_partition(a, b, n, i + 1), u)
                        u <= uniform_partition(a, b, n, i + 1)
                        interval_contains_left(uniform_partition(a, b, n, i), uniform_partition(a, b, n, i + 1), v)
                        uniform_partition(a, b, n, i) <= v
                        interval_contains_right(uniform_partition(a, b, n, i), uniform_partition(a, b, n, i + 1), v)
                        v <= uniform_partition(a, b, n, i + 1)
                        partition_point_in_interval(uniform_partition(a, b, n), a, b, n.suc, i)
                        interval_contains(a, b, uniform_partition(a, b, n, i))
                        partition_point_in_interval(uniform_partition(a, b, n), a, b, n.suc, i + 1)
                        interval_contains(a, b, uniform_partition(a, b, n, i + 1))
                        interval_contains_mono(a, b, uniform_partition(a, b, n, i),
                            uniform_partition(a, b, n, i + 1), u)
                        interval_contains(a, b, u)
                        interval_contains_mono(a, b, uniform_partition(a, b, n, i),
                            uniform_partition(a, b, n, i + 1), v)
                        interval_contains(a, b, v)
                        forall(u0: Real, v0: Real) {
                            interval_contains(a, b, u0) and interval_contains(a, b, v0) implies (f(u0) - f(v0)).abs <= c * (u0 - v0).abs
                        }
                        interval_contains(a, b, u) and interval_contains(a, b, v) implies (f(u) - f(v)).abs <= c * (u - v).abs
                        interval_contains(a, b, u) and interval_contains(a, b, v)
                        (f(u) - f(v)).abs <= c * (u - v).abs
                        interval_abs_diff_le_width(uniform_partition(a, b, n, i),
                            uniform_partition(a, b, n, i + 1), u, v)
                        (u - v).abs <= uniform_partition(a, b, n, i + 1) - uniform_partition(a, b, n, i)
                        (u - v).abs <= uniform_mesh(a, b, n)
                        Real.0 <= c
                        mul_le_mul_of_nonneg_right((u - v).abs, uniform_mesh(a, b, n), c)
                        (u - v).abs * c <= uniform_mesh(a, b, n) * c
                        real_mul_comm((u - v).abs, c)
                        (u - v).abs * c = c * (u - v).abs
                        real_mul_comm(uniform_mesh(a, b, n), c)
                        uniform_mesh(a, b, n) * c = c * uniform_mesh(a, b, n)
                        c * (u - v).abs <= c * uniform_mesh(a, b, n)
                        lte_trans[Real]((f(u) - f(v)).abs, c * (u - v).abs, c * uniform_mesh(a, b, n))
                        (f(u) - f(v)).abs <= c * uniform_mesh(a, b, n)
                        uniform_partition(a, b, n, i + 1) - uniform_partition(a, b, n, i) =
                            uniform_mesh(a, b, n)
                        c * (uniform_partition(a, b, n, i + 1) - uniform_partition(a, b, n, i)) =
                            c * uniform_mesh(a, b, n)
                        (f(u) - f(v)).abs <= c * (uniform_partition(a, b, n, i + 1) - uniform_partition(a, b, n, i))
                    }
                }
                interval_sup_sub_inf_le_lipschitz_gen(f, c,
                    uniform_partition(a, b, n, i), uniform_partition(a, b, n, i + 1))
                interval_sup(f, uniform_partition(a, b, n, i), uniform_partition(a, b, n, i + 1)) -
                    interval_inf(f, uniform_partition(a, b, n, i), uniform_partition(a, b, n, i + 1)) <= c * (uniform_partition(a, b, n, i + 1) - uniform_partition(a, b, n, i))
                interval_sup(f, uniform_partition(a, b, n, i), uniform_partition(a, b, n, i + 1)) -
                    interval_inf(f, uniform_partition(a, b, n, i), uniform_partition(a, b, n, i + 1)) <= c * uniform_mesh(a, b, n)
                partition_step_upper(f, uniform_partition(a, b, n), i) =
                    interval_sup(f, uniform_partition(a, b, n, i), uniform_partition(a, b, n, i + 1)) *
                        diff_step(uniform_partition(a, b, n), i)
                diff_step(uniform_partition(a, b, n), i) =
                    uniform_partition(a, b, n, i + 1) - uniform_partition(a, b, n, i)
                diff_step(uniform_partition(a, b, n), i) = uniform_mesh(a, b, n)
                partition_step_upper(f, uniform_partition(a, b, n), i) =
                    interval_sup(f, uniform_partition(a, b, n, i), uniform_partition(a, b, n, i + 1)) *
                        uniform_mesh(a, b, n)
                partition_step_lower(f, uniform_partition(a, b, n), i) =
                    interval_inf(f, uniform_partition(a, b, n, i), uniform_partition(a, b, n, i + 1)) *
                        diff_step(uniform_partition(a, b, n), i)
                partition_step_lower(f, uniform_partition(a, b, n), i) =
                    interval_inf(f, uniform_partition(a, b, n, i), uniform_partition(a, b, n, i + 1)) *
                        uniform_mesh(a, b, n)
                partition_step_upper(f, uniform_partition(a, b, n), i) -
                    partition_step_lower(f, uniform_partition(a, b, n), i) =
                    (interval_sup(f, uniform_partition(a, b, n, i), uniform_partition(a, b, n, i + 1)) -
                        interval_inf(f, uniform_partition(a, b, n, i), uniform_partition(a, b, n, i + 1))) *
                        uniform_mesh(a, b, n)
                from_nat_suc_pos_real(n)
                from_nat[Real](n.suc) > Real.0
                from_nat[Real](n.suc) != Real.0
                sub_nonneg(a, b)
                Real.0 <= b - a
                div_nonneg_pos_denom(b - a, from_nat[Real](n.suc))
                Real.0 <= uniform_mesh(a, b, n)
                mul_le_mul_of_nonneg_right(
                    interval_sup(f, uniform_partition(a, b, n, i), uniform_partition(a, b, n, i + 1)) -
                        interval_inf(f, uniform_partition(a, b, n, i), uniform_partition(a, b, n, i + 1)),
                    c * uniform_mesh(a, b, n), uniform_mesh(a, b, n))
                (interval_sup(f, uniform_partition(a, b, n, i), uniform_partition(a, b, n, i + 1)) -
                    interval_inf(f, uniform_partition(a, b, n, i), uniform_partition(a, b, n, i + 1))) *
                    uniform_mesh(a, b, n) <= (c * uniform_mesh(a, b, n)) * uniform_mesh(a, b, n)
                partition_step_upper(f, uniform_partition(a, b, n), i) -
                    partition_step_lower(f, uniform_partition(a, b, n), i) <= (c * uniform_mesh(a, b, n)) * uniform_mesh(a, b, n)
                sub_seq(partition_step_upper(f, uniform_partition(a, b, n)),
                    partition_step_lower(f, uniform_partition(a, b, n)), i) =
                    partition_step_upper(f, uniform_partition(a, b, n), i) -
                    partition_step_lower(f, uniform_partition(a, b, n), i)
                sub_seq(partition_step_upper(f, uniform_partition(a, b, n)),
                    partition_step_lower(f, uniform_partition(a, b, n)), i) <= (c * uniform_mesh(a, b, n)) * uniform_mesh(a, b, n)
            }
        }
        partial_lte(sub_seq(partition_step_upper(f, uniform_partition(a, b, n)),
                partition_step_lower(f, uniform_partition(a, b, n))),
            const_seq((c * uniform_mesh(a, b, n)) * uniform_mesh(a, b, n)), n.suc)
        partial(sub_seq(partition_step_upper(f, uniform_partition(a, b, n)),
                partition_step_lower(f, uniform_partition(a, b, n))), n.suc) <= partial(const_seq((c * uniform_mesh(a, b, n)) * uniform_mesh(a, b, n)), n.suc)
        forall(j: Nat) {
            if j < n.suc {
                const_seq((c * uniform_mesh(a, b, n)) * uniform_mesh(a, b, n), j) =
                    (c * uniform_mesh(a, b, n)) * uniform_mesh(a, b, n)
            }
        }
        partial_const(const_seq((c * uniform_mesh(a, b, n)) * uniform_mesh(a, b, n)), n.suc,
            (c * uniform_mesh(a, b, n)) * uniform_mesh(a, b, n))
        partial(const_seq((c * uniform_mesh(a, b, n)) * uniform_mesh(a, b, n)), n.suc) =
            from_nat[Real](n.suc) * ((c * uniform_mesh(a, b, n)) * uniform_mesh(a, b, n))
        partial(sub_seq(partition_step_upper(f, uniform_partition(a, b, n)),
                partition_step_lower(f, uniform_partition(a, b, n))), n.suc) <= from_nat[Real](n.suc) * ((c * uniform_mesh(a, b, n)) * uniform_mesh(a, b, n))
        partial_sub_seq(partition_step_upper(f, uniform_partition(a, b, n)),
            partition_step_lower(f, uniform_partition(a, b, n)), n.suc)
        partial(sub_seq(partition_step_upper(f, uniform_partition(a, b, n)),
                partition_step_lower(f, uniform_partition(a, b, n))), n.suc) =
            partial(partition_step_upper(f, uniform_partition(a, b, n)), n.suc) -
            partial(partition_step_lower(f, uniform_partition(a, b, n)), n.suc)
        partial(partition_step_upper(f, uniform_partition(a, b, n)), n.suc) -
            partial(partition_step_lower(f, uniform_partition(a, b, n)), n.suc) <= from_nat[Real](n.suc) * ((c * uniform_mesh(a, b, n)) * uniform_mesh(a, b, n))
        from_nat_suc_pos_real(n)
        from_nat[Real](n.suc) > Real.0
        from_nat[Real](n.suc) != Real.0
        div_mul_cancel_denominator(b - a, from_nat[Real](n.suc))
        ((b - a) / from_nat[Real](n.suc)) * from_nat[Real](n.suc) = b - a
        uniform_mesh(a, b, n) * from_nat[Real](n.suc) = b - a
        // Simplify from_nat * (c * mesh * mesh) to c * (b - a) * (b - a) / from_nat.
        from_nat[Real](n.suc) * ((c * uniform_mesh(a, b, n)) * uniform_mesh(a, b, n)) =
            c * (from_nat[Real](n.suc) * (uniform_mesh(a, b, n) * uniform_mesh(a, b, n)))
        from_nat[Real](n.suc) * (uniform_mesh(a, b, n) * uniform_mesh(a, b, n)) =
            (from_nat[Real](n.suc) * uniform_mesh(a, b, n)) * uniform_mesh(a, b, n)
        real_mul_comm(uniform_mesh(a, b, n), from_nat[Real](n.suc))
        from_nat[Real](n.suc) * uniform_mesh(a, b, n) = uniform_mesh(a, b, n) * from_nat[Real](n.suc)
        from_nat[Real](n.suc) * uniform_mesh(a, b, n) = b - a
        from_nat[Real](n.suc) * (uniform_mesh(a, b, n) * uniform_mesh(a, b, n)) =
            (b - a) * uniform_mesh(a, b, n)
        c * (from_nat[Real](n.suc) * (uniform_mesh(a, b, n) * uniform_mesh(a, b, n))) =
            c * ((b - a) * uniform_mesh(a, b, n))
        c * ((b - a) * uniform_mesh(a, b, n)) = (c * (b - a)) * uniform_mesh(a, b, n)
        from_nat[Real](n.suc) * ((c * uniform_mesh(a, b, n)) * uniform_mesh(a, b, n)) =
            (c * (b - a)) * uniform_mesh(a, b, n)
        real_mul_comm(c * (b - a), uniform_mesh(a, b, n))
        (c * (b - a)) * uniform_mesh(a, b, n) = uniform_mesh(a, b, n) * (c * (b - a))
        uniform_mesh(a, b, n) * (c * (b - a)) = ((b - a) / from_nat[Real](n.suc)) * (c * (b - a))
        mul_frac_right(b - a, from_nat[Real](n.suc), c * (b - a))
        ((b - a) / from_nat[Real](n.suc)) * (c * (b - a)) = ((b - a) * (c * (b - a))) / from_nat[Real](n.suc)
        uniform_mesh(a, b, n) * (c * (b - a)) = ((b - a) * (c * (b - a))) / from_nat[Real](n.suc)
        real_mul_comm(b - a, c * (b - a))
        (b - a) * (c * (b - a)) = (c * (b - a)) * (b - a)
        uniform_mesh(a, b, n) * (c * (b - a)) = ((c * (b - a)) * (b - a)) / from_nat[Real](n.suc)
        (c * (b - a)) * uniform_mesh(a, b, n) = ((c * (b - a)) * (b - a)) / from_nat[Real](n.suc)
        from_nat[Real](n.suc) * ((c * uniform_mesh(a, b, n)) * uniform_mesh(a, b, n)) =
            ((c * (b - a)) * (b - a)) / from_nat[Real](n.suc)
        partial(partition_step_upper(f, uniform_partition(a, b, n)), n.suc) -
            partial(partition_step_lower(f, uniform_partition(a, b, n)), n.suc) <= ((c * (b - a)) * (b - a)) / from_nat[Real](n.suc)
        upper_sum(f, uniform_partition(a, b, n), n.suc) =
            partial(partition_step_upper(f, uniform_partition(a, b, n)), n.suc)
        lower_sum(f, uniform_partition(a, b, n), n.suc) =
            partial(partition_step_lower(f, uniform_partition(a, b, n)), n.suc)
        upper_sum(f, uniform_partition(a, b, n), n.suc) -
            lower_sum(f, uniform_partition(a, b, n), n.suc) <= ((c * (b - a)) * (b - a)) / from_nat[Real](n.suc)
    }
}

/// A c-Lipschitz f bounded by m in absolute value on [a, b], with known
/// antiderivative g, is integrable on [a, b].
theorem fn_integrable_gen_sym(f: Real -> Real, g: Real -> Real, c: Real, m: Real, a: Real, b: Real) {
    a <= b and Real.0 <= c and continuous(g) and is_derivative_fn(g, f) and
    (forall(u: Real, v: Real) {
        interval_contains(a, b, u) and interval_contains(a, b, v) implies (f(u) - f(v)).abs <= c * (u - v).abs
    }) and
    (forall(t: Real) { interval_contains(a, b, t) implies -m <= f(t) }) and
    (forall(t: Real) { interval_contains(a, b, t) implies f(t) <= m })
    implies
    is_integrable(f, a, b)
} by {
    if a <= b and Real.0 <= c and continuous(g) and is_derivative_fn(g, f) and
       (forall(u: Real, v: Real) {
           interval_contains(a, b, u) and interval_contains(a, b, v) implies (f(u) - f(v)).abs <= c * (u - v).abs
       }) and
       (forall(t: Real) { interval_contains(a, b, t) implies -m <= f(t) }) and
       (forall(t: Real) { interval_contains(a, b, t) implies f(t) <= m }) {
        fn_lower_sum_set_sup_exists(f, g, a, b, -m)
        let l: Real satisfy {
            is_set_supremum(lower_sum_set(f, a, b), l)
        }
        fn_upper_sum_set_inf_exists(f, g, a, b, m)
        let u: Real satisfy {
            is_set_infimum(upper_sum_set(f, a, b), u)
        }
        fn_lower_sum_set_bounded_above(f, g, a, b, -m)
        is_set_upper_bound(lower_sum_set(f, a, b), g(b) - g(a))
        set_supremum_le_upper_bound(lower_sum_set(f, a, b), l, g(b) - g(a))
        l <= g(b) - g(a)
        fn_upper_sum_set_bounded_below(f, g, a, b, m)
        is_set_lower_bound(upper_sum_set(f, a, b), g(b) - g(a))
        set_lower_bound_le_infimum(upper_sum_set(f, a, b), u, g(b) - g(a))
        g(b) - g(a) <= u
        lte_trans[Real](l, g(b) - g(a), u)
        l <= u
        sub_nonneg(l, u)
        Real.0 <= u - l
        // For every n the uniform partition forces u - l below c * (b - a)^2 / (n + 1).
        forall(n: Nat) {
            lipschitz_uniform_upper_minus_lower_gen(f, c, m, a, b, n)
            upper_sum(f, uniform_partition(a, b, n), n.suc) -
                lower_sum(f, uniform_partition(a, b, n), n.suc) <= (c * (b - a) * (b - a)) / from_nat[Real](n.suc)
            uniform_partition_is_partition(a, b, n)
            is_partition(uniform_partition(a, b, n), a, b, n.suc)
            is_partition(uniform_partition(a, b, n), a, b, n.suc) and
                lower_sum(f, uniform_partition(a, b, n), n.suc) =
                    lower_sum(f, uniform_partition(a, b, n), n.suc)
            exists(p0: Nat -> Real, n0: Nat) {
                is_partition(p0, a, b, n0) and
                    lower_sum(f, uniform_partition(a, b, n), n.suc) = lower_sum(f, p0, n0)
            }
            lower_sum_contains(f, a, b, lower_sum(f, uniform_partition(a, b, n), n.suc))
            lower_sum_set(f, a, b).contains(lower_sum(f, uniform_partition(a, b, n), n.suc)) =
                lower_sum_contains(f, a, b, lower_sum(f, uniform_partition(a, b, n), n.suc))
            lower_sum_set(f, a, b).contains(lower_sum(f, uniform_partition(a, b, n), n.suc))
            set_member_le_supremum(lower_sum_set(f, a, b), l,
                lower_sum(f, uniform_partition(a, b, n), n.suc))
            lower_sum(f, uniform_partition(a, b, n), n.suc) <= l
            is_partition(uniform_partition(a, b, n), a, b, n.suc) and
                upper_sum(f, uniform_partition(a, b, n), n.suc) =
                    upper_sum(f, uniform_partition(a, b, n), n.suc)
            exists(p1: Nat -> Real, n1: Nat) {
                is_partition(p1, a, b, n1) and
                    upper_sum(f, uniform_partition(a, b, n), n.suc) = upper_sum(f, p1, n1)
            }
            upper_sum_contains(f, a, b, upper_sum(f, uniform_partition(a, b, n), n.suc))
            upper_sum_set(f, a, b).contains(upper_sum(f, uniform_partition(a, b, n), n.suc)) =
                upper_sum_contains(f, a, b, upper_sum(f, uniform_partition(a, b, n), n.suc))
            upper_sum_set(f, a, b).contains(upper_sum(f, uniform_partition(a, b, n), n.suc))
            set_infimum_is_lower_bound(upper_sum_set(f, a, b), u)
            is_set_lower_bound(upper_sum_set(f, a, b), u)
            set_lower_bound_contains_le(upper_sum_set(f, a, b), u,
                upper_sum(f, uniform_partition(a, b, n), n.suc))
            u <= upper_sum(f, uniform_partition(a, b, n), n.suc)
            neg_lte_flip(lower_sum(f, uniform_partition(a, b, n), n.suc), l)
            -l <= -lower_sum(f, uniform_partition(a, b, n), n.suc)
            add_le_add(u, upper_sum(f, uniform_partition(a, b, n), n.suc),
                -l, -lower_sum(f, uniform_partition(a, b, n), n.suc))
            u + -l <= upper_sum(f, uniform_partition(a, b, n), n.suc) +
                -lower_sum(f, uniform_partition(a, b, n), n.suc)
            u - l = u + -l
            upper_sum(f, uniform_partition(a, b, n), n.suc) -
                lower_sum(f, uniform_partition(a, b, n), n.suc) =
                upper_sum(f, uniform_partition(a, b, n), n.suc) +
                -lower_sum(f, uniform_partition(a, b, n), n.suc)
            u - l <= upper_sum(f, uniform_partition(a, b, n), n.suc) -
                lower_sum(f, uniform_partition(a, b, n), n.suc)
            lte_trans[Real](u - l,
                upper_sum(f, uniform_partition(a, b, n), n.suc) -
                    lower_sum(f, uniform_partition(a, b, n), n.suc),
                (c * (b - a) * (b - a)) / from_nat[Real](n.suc))
            u - l <= (c * (b - a) * (b - a)) / from_nat[Real](n.suc)
        }
        nonneg_frac_all_n_imp_zero(c * (b - a) * (b - a), u - l)
        u - l = Real.0
        sub_zero_imp_eq(u, l)
        u = l
        is_set_infimum(upper_sum_set(f, a, b), l)
        is_set_supremum(lower_sum_set(f, a, b), l) and is_set_infimum(upper_sum_set(f, a, b), l)
        exists(w: Real) {
            is_set_supremum(lower_sum_set(f, a, b), w) and is_set_infimum(upper_sum_set(f, a, b), w)
        }
        is_integrable(f, a, b)
    }
}

// ---------------------------------------------------------------------------
// The square function on the unit interval
// ---------------------------------------------------------------------------

/// The square function is two-Lipschitz on [0, 1].
theorem square_lipschitz_two_on_unit {
    Real.0 <= Real.1 implies forall(u: Real, v: Real) {
        interval_contains(Real.0, Real.1, u) and interval_contains(Real.0, Real.1, v)
        implies (square_real(u) - square_real(v)).abs <= two * (u - v).abs
    }
} by {
    if Real.0 <= Real.1 {
        forall(u: Real, v: Real) {
            if interval_contains(Real.0, Real.1, u) and interval_contains(Real.0, Real.1, v) {
                interval_contains_left(Real.0, Real.1, u)
                Real.0 <= u
                interval_contains_right(Real.0, Real.1, u)
                u <= Real.1
                interval_contains_left(Real.0, Real.1, v)
                Real.0 <= v
                interval_contains_right(Real.0, Real.1, v)
                v <= Real.1
                // square(u) - square(v) = (u - v) * (u + v)
                square_real(u) = u * u
                square_real(v) = v * v
                square_real(u) - square_real(v) = u * u - v * v
                mul_sub_distrib_left(u, v, u + v)
                (u - v) * (u + v) = u * (u + v) - v * (u + v)
                mul_distrib_left(u, u, v)
                u * (u + v) = u * u + u * v
                mul_distrib_left(v, u, v)
                v * (u + v) = v * u + v * v
                real_mul_comm(v, u)
                v * u = u * v
                u * (u + v) - v * (u + v) = (u * u + u * v) - (u * v + v * v)
                u * u - v * v = (u * u + u * v) - (u * v + v * v)
                (u - v) * (u + v) = u * u - v * v
                (u - v) * (u + v) = square_real(u) - square_real(v)
                mul_abs(u - v, u + v)
                ((u - v) * (u + v)).abs = (u - v).abs * (u + v).abs
                (square_real(u) - square_real(v)).abs = (u - v).abs * (u + v).abs
                // u + v >= 0
                add_le_add[Real](Real.0, u, Real.0, v)
                Real.0 + Real.0 <= u + v
                Real.0 + Real.0 = Real.0
                Real.0 <= u + v
                abs_of_nonneg(u + v)
                (u + v).abs = u + v
                (square_real(u) - square_real(v)).abs = (u - v).abs * (u + v)
                // u + v <= two
                add_le_add[Real](u, Real.1, v, Real.1)
                u + v <= Real.1 + Real.1
                two = Real.1 + Real.1
                u + v <= two
                abs_gte_zero(u - v)
                Real.0 <= (u - v).abs
                mul_le_mul_of_nonneg_right(u + v, two, (u - v).abs)
                (u + v) * (u - v).abs <= two * (u - v).abs
                real_mul_comm(u + v, (u - v).abs)
                (u + v) * (u - v).abs = (u - v).abs * (u + v)
                (u - v).abs * (u + v) <= two * (u - v).abs
                (square_real(u) - square_real(v)).abs <= two * (u - v).abs
            }
        }
    }
}

/// The square function is bounded below by -1 on [0, 1].
theorem square_lower_bound_on_unit {
    Real.0 <= Real.1 implies forall(t: Real) {
        interval_contains(Real.0, Real.1, t) implies -Real.1 <= square_real(t)
    }
} by {
    if Real.0 <= Real.1 {
        forall(t: Real) {
            if interval_contains(Real.0, Real.1, t) {
                interval_contains_left(Real.0, Real.1, t)
                Real.0 <= t
                interval_contains_right(Real.0, Real.1, t)
                t <= Real.1
                square_real(t) = t * t
                square_nonneg(t)
                t * t >= Real.0
                Real.0 <= square_real(t)
                lte_imp_neg_lte_neg(Real.0, Real.1)
                -Real.1 <= -Real.0
                neg_zero
                -Real.0 = Real.0
                -Real.1 <= Real.0
                lte_trans[Real](-Real.1, Real.0, square_real(t))
                -Real.1 <= square_real(t)
            }
        }
    }
}

/// The square function is bounded above by 1 on [0, 1].
theorem square_upper_bound_on_unit {
    Real.0 <= Real.1 implies forall(t: Real) {
        interval_contains(Real.0, Real.1, t) implies square_real(t) <= Real.1
    }
} by {
    if Real.0 <= Real.1 {
        forall(t: Real) {
            if interval_contains(Real.0, Real.1, t) {
                interval_contains_left(Real.0, Real.1, t)
                Real.0 <= t
                interval_contains_right(Real.0, Real.1, t)
                t <= Real.1
                square_real(t) = t * t
                sq_lte_base(t)
                t.pow(Nat.2) <= t
                pow_suc(t, Nat.1)
                t.pow(Nat.1.suc) = t * t.pow(Nat.1)
                pow_one[Real](t)
                t.pow(Nat.1) = t
                Nat.1.suc = Nat.2
                t.pow(Nat.2) = t * t
                t * t <= t
                square_real(t) <= t
                lte_trans[Real](square_real(t), t, Real.1)
                square_real(t) <= Real.1
            }
        }
    }
}

// ---------------------------------------------------------------------------
// The integral of x^2 over [0, 1]
// ---------------------------------------------------------------------------

/// The integral of the square function over [0, 1] is 1/3.
///
/// The antiderivative is x^3 / 3; the square is two-Lipschitz and bounded by
/// one in absolute value on [0, 1], so fn_integrable_gen_sym applies and
/// ftc2_general gives integral(x^2, 0, 1) = 1/3 - 0.
theorem integral_square_unit {
    integral(square_real, Real.0, Real.1) = one_third
} by {
    Real.1 > Real.0
    lt_imp_lte(Real.0, Real.1)
    Real.0 <= Real.1
    if Real.0 <= Real.1 {
        // The square is two-Lipschitz and bounded by one in absolute value on [0, 1].
        square_lipschitz_two_on_unit
        square_lower_bound_on_unit
        square_upper_bound_on_unit
        // The Lipschitz constant two is nonnegative.
        two_positive
        two > Real.0
        lt_imp_lte(Real.0, two)
        Real.0 <= two
        // The antiderivative x^3 / 3 is continuous with derivative the square.
        third_cube_is_derivative_fn
        is_derivative_fn(pointwise_div_real(cube_real, constant[Real, Real](three)), square_real)
        is_derivative_fn_imp_continuous(pointwise_div_real(cube_real, constant[Real, Real](three)), square_real)
        continuous(pointwise_div_real(cube_real, constant[Real, Real](three)))
        // Integrability of the square on [0, 1].
        fn_integrable_gen_sym(square_real, pointwise_div_real(cube_real, constant[Real, Real](three)),
            two, Real.1, Real.0, Real.1)
        is_integrable(square_real, Real.0, Real.1)
        // The fundamental theorem of calculus forces the value.
        ftc2_general(square_real, pointwise_div_real(cube_real, constant[Real, Real](three)),
            Real.0, Real.1, -Real.1, Real.1)
        integral(square_real, Real.0, Real.1) =
            pointwise_div_real(cube_real, constant[Real, Real](three), Real.1) -
            pointwise_div_real(cube_real, constant[Real, Real](three), Real.0)
        // The antiderivative at 1 is 1/3.
        pointwise_div_real(cube_real, constant[Real, Real](three), Real.1) =
            cube_real(Real.1) / constant[Real, Real](three, Real.1)
        constant[Real, Real](three, Real.1) = three
        cube_real(Real.1) = Real.1
        pointwise_div_real(cube_real, constant[Real, Real](three), Real.1) = Real.1 / three
        one_third = Real.1 / three
        pointwise_div_real(cube_real, constant[Real, Real](three), Real.1) = one_third
        // The antiderivative at 0 is 0.
        pointwise_div_real(cube_real, constant[Real, Real](three), Real.0) =
            cube_real(Real.0) / constant[Real, Real](three, Real.0)
        constant[Real, Real](three, Real.0) = three
        cube_real(Real.0) = Real.0
        pointwise_div_real(cube_real, constant[Real, Real](three), Real.0) = Real.0 / three
        Real.0 / three = Real.0
        pointwise_div_real(cube_real, constant[Real, Real](three), Real.0) = Real.0
        // 1/3 - 0 = 1/3.
        one_third - Real.0 = one_third
        integral(square_real, Real.0, Real.1) = one_third
    }
}
