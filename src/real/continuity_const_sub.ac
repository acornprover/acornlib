from data.basic.function_algebra import pointwise_neg
from real.continuity_const_add import const_add_left, const_add_right,
    continuous_at_const_add_left, continuous_at_const_add_right,
    continuous_const_add_left, continuous_const_add_right
from real.continuity_pointwise import continuous_at_pointwise_neg,
    continuous_pointwise_neg
from real.continuity_base import Real, continuous, continuous_at

/// The function obtained by subtracting f(x) from the fixed real constant c.
define const_sub_left(c: Real, f: Real -> Real, x: Real) -> Real {
    c - f(x)
}

/// The function obtained by subtracting the fixed real constant c from f(x).
define const_sub_right(f: Real -> Real, c: Real, x: Real) -> Real {
    f(x) - c
}

/// Subtraction on the left from a fixed constant agrees with adding the constant on the left to the pointwise negation.
theorem const_sub_left_eq_const_add_left_neg(c: Real, f: Real -> Real) {
    const_sub_left(c, f) = const_add_left(c, pointwise_neg[Real, Real](f))
} by {
    forall(x: Real) {
        pointwise_neg[Real, Real](f, x) = -f(x)
        const_add_left(c, pointwise_neg[Real, Real](f), x) = c + (-f(x))
        c + (-f(x)) = c - f(x)
        const_sub_left(c, f, x) = c - f(x)
        const_sub_left(c, f, x) = const_add_left(c, pointwise_neg[Real, Real](f), x)
    }
}

/// Subtraction of a fixed constant on the right agrees with adding the negated constant on the right.
theorem const_sub_right_eq_const_add_right_neg(f: Real -> Real, c: Real) {
    const_sub_right(f, c) = const_add_right(f, -c)
} by {
    forall(x: Real) {
        const_add_right(f, -c, x) = f(x) + (-c)
        f(x) + (-c) = f(x) - c
        const_sub_right(f, c, x) = f(x) - c
        const_sub_right(f, c, x) = const_add_right(f, -c, x)
    }
}

/// Subtraction from a fixed real constant on the left preserves continuity at a point.
theorem continuous_at_const_sub_left(c: Real, f: Real -> Real, x: Real) {
    continuous_at(f, x)
    implies continuous_at(const_sub_left(c, f), x)
} by {
    continuous_at_pointwise_neg(f, x)
    continuous_at(pointwise_neg[Real, Real](f), x)
    continuous_at_const_add_left(c, pointwise_neg[Real, Real](f), x)
    continuous_at(const_add_left(c, pointwise_neg[Real, Real](f)), x)
    const_sub_left_eq_const_add_left_neg(c, f)
    continuous_at(const_sub_left(c, f), x)
}

/// Subtraction from a fixed real constant on the left preserves continuous functions.
theorem continuous_const_sub_left(c: Real, f: Real -> Real) {
    continuous(f) implies continuous(const_sub_left(c, f))
} by {
    continuous_pointwise_neg(f)
    continuous(pointwise_neg[Real, Real](f))
    continuous_const_add_left(c, pointwise_neg[Real, Real](f))
    continuous(const_add_left(c, pointwise_neg[Real, Real](f)))
    const_sub_left_eq_const_add_left_neg(c, f)
    continuous(const_sub_left(c, f))
}

/// Subtraction of a fixed real constant on the right preserves continuity at a point.
theorem continuous_at_const_sub_right(f: Real -> Real, c: Real, x: Real) {
    continuous_at(f, x)
    implies continuous_at(const_sub_right(f, c), x)
} by {
    continuous_at_const_add_right(f, -c, x)
    continuous_at(const_add_right(f, -c), x)
    const_sub_right_eq_const_add_right_neg(f, c)
    continuous_at(const_sub_right(f, c), x)
}

/// Subtraction of a fixed real constant on the right preserves continuous functions.
theorem continuous_const_sub_right(f: Real -> Real, c: Real) {
    continuous(f) implies continuous(const_sub_right(f, c))
} by {
    continuous_const_add_right(f, -c)
    continuous(const_add_right(f, -c))
    const_sub_right_eq_const_add_right_neg(f, c)
    continuous(const_sub_right(f, c))
}
