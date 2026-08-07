/// Conservative derivative corollaries for reciprocal and quotient functions.

from data.basic.functions import compose
from real.continuity_base import continuous_at
from real.derivative_basic import has_derivative_at, differentiable_at
from real.derivative_chain import derivative_compose
from real.derivative_continuity import derivative_continuous_at,
    differentiable_continuous_at
from real.derivative_quotient import reciprocal_real, pointwise_reciprocal_real,
    pointwise_div_real, derivative_reciprocal_real,
    derivative_pointwise_reciprocal_real, differentiable_pointwise_reciprocal_real,
    derivative_pointwise_div, differentiable_pointwise_div
from real.real_base import Real

/// Composing the reciprocal function after `g` has the chain-rule reciprocal derivative.
theorem derivative_compose_reciprocal_real(g: Real -> Real, x0: Real, dg: Real) {
    has_derivative_at(g, x0, dg) and g(x0) != Real.0 implies
        has_derivative_at(compose(reciprocal_real, g), x0,
            (-Real.1 / (g(x0) * g(x0))) * dg)
} by {
    if has_derivative_at(g, x0, dg) and g(x0) != Real.0 {
        derivative_reciprocal_real(g(x0))
        has_derivative_at(reciprocal_real, g(x0), -Real.1 / (g(x0) * g(x0)))
        derivative_compose(g, reciprocal_real, x0, dg, -Real.1 / (g(x0) * g(x0)))
        has_derivative_at(compose(reciprocal_real, g), x0,
            (-Real.1 / (g(x0) * g(x0))) * dg)
    }
}

/// Composing the reciprocal function after a differentiable nonzero-valued function is differentiable.
theorem differentiable_compose_reciprocal_real(g: Real -> Real, x0: Real) {
    differentiable_at(g, x0) and g(x0) != Real.0
        implies differentiable_at(compose(reciprocal_real, g), x0)
} by {
    if differentiable_at(g, x0) and g(x0) != Real.0 {
        let dg: Real satisfy {
            has_derivative_at(g, x0, dg)
        }
        derivative_compose_reciprocal_real(g, x0, dg)
        has_derivative_at(compose(reciprocal_real, g), x0,
            (-Real.1 / (g(x0) * g(x0))) * dg)
        exists(d: Real) {
            has_derivative_at(compose(reciprocal_real, g), x0, d)
        }
    }
}

/// A differentiable reciprocal transform is continuous at the point of differentiation.
theorem derivative_pointwise_reciprocal_real_continuous_at(g: Real -> Real, x0: Real, dg: Real) {
    has_derivative_at(g, x0, dg) and g(x0) != Real.0
        implies continuous_at(pointwise_reciprocal_real(g), x0)
} by {
    if has_derivative_at(g, x0, dg) and g(x0) != Real.0 {
        derivative_pointwise_reciprocal_real(g, x0, dg)
        has_derivative_at(pointwise_reciprocal_real(g), x0,
            (-Real.1 / (g(x0) * g(x0))) * dg)
        derivative_continuous_at(pointwise_reciprocal_real(g), x0,
            (-Real.1 / (g(x0) * g(x0))) * dg)
        continuous_at(pointwise_reciprocal_real(g), x0)
    }
}

/// A reciprocal of a differentiable nonzero-valued function is continuous at that point.
theorem differentiable_pointwise_reciprocal_real_continuous_at(g: Real -> Real, x0: Real) {
    differentiable_at(g, x0) and g(x0) != Real.0
        implies continuous_at(pointwise_reciprocal_real(g), x0)
} by {
    if differentiable_at(g, x0) and g(x0) != Real.0 {
        differentiable_pointwise_reciprocal_real(g, x0)
        differentiable_at(pointwise_reciprocal_real(g), x0)
        differentiable_continuous_at(pointwise_reciprocal_real(g), x0)
        continuous_at(pointwise_reciprocal_real(g), x0)
    }
}

/// A quotient with differentiable numerator and denominator is continuous at nonzero denominator values.
theorem derivative_pointwise_div_continuous_at(
    f: Real -> Real, g: Real -> Real, x0: Real, df: Real, dg: Real
) {
    has_derivative_at(f, x0, df) and has_derivative_at(g, x0, dg) and g(x0) != Real.0
        implies continuous_at(pointwise_div_real(f, g), x0)
} by {
    if has_derivative_at(f, x0, df) and has_derivative_at(g, x0, dg) and g(x0) != Real.0 {
        derivative_pointwise_div(f, g, x0, df, dg)
        has_derivative_at(pointwise_div_real(f, g), x0,
            (df * g(x0) - f(x0) * dg) / (g(x0) * g(x0)))
        derivative_continuous_at(pointwise_div_real(f, g), x0,
            (df * g(x0) - f(x0) * dg) / (g(x0) * g(x0)))
        continuous_at(pointwise_div_real(f, g), x0)
    }
}

/// A quotient of differentiable functions is continuous at nonzero denominator values.
theorem differentiable_pointwise_div_continuous_at(f: Real -> Real, g: Real -> Real, x0: Real) {
    differentiable_at(f, x0) and differentiable_at(g, x0) and g(x0) != Real.0
        implies continuous_at(pointwise_div_real(f, g), x0)
} by {
    if differentiable_at(f, x0) and differentiable_at(g, x0) and g(x0) != Real.0 {
        differentiable_pointwise_div(f, g, x0)
        differentiable_at(pointwise_div_real(f, g), x0)
        differentiable_continuous_at(pointwise_div_real(f, g), x0)
        continuous_at(pointwise_div_real(f, g), x0)
    }
}
