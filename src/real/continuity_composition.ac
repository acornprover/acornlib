from data.basic.functions import compose, constant_is_constant
from real.continuity_base import Real, continuous, continuous_at, continuous_condition,
    constant_is_uniform, uniform, uniform_imp_continuous

/// Introduces continuity at a point from the epsilon-delta condition.
theorem continuous_at_intro(f: Real -> Real, x: Real) {
    forall(eps: Real) {
        eps.is_positive implies exists(delta: Real) {
            delta.is_positive and continuous_condition(f, x, delta, eps)
        }
    } implies continuous_at(f, x)
}

/// Introduces continuity from pointwise continuity.
theorem continuous_intro(f: Real -> Real) {
    forall(x: Real) {
        continuous_at(f, x)
    } implies continuous(f)
}

/// A continuous real function is continuous at each real point.
theorem continuous_imp_continuous_at(f: Real -> Real, x: Real) {
    continuous(f) implies continuous_at(f, x)
} by {
    continuous(f) = forall(y: Real) {
        continuous_at(f, y)
    }
    continuous_at(f, x)
}

/// A positive tolerance for a continuous function admits a local continuity radius.
theorem continuous_at_delta(f: Real -> Real, x: Real, eps: Real) {
    continuous_at(f, x) and eps.is_positive implies exists(delta: Real) {
        delta.is_positive and continuous_condition(f, x, delta, eps)
    }
} by {
    continuous_at(f, x) = forall(eps2: Real) {
        eps2.is_positive implies exists(delta: Real) {
            delta.is_positive and continuous_condition(f, x, delta, eps2)
        }
    }
    exists(delta2: Real) {
        delta2.is_positive and continuous_condition(f, x, delta2, eps)
    }
}

/// A constant real function is uniformly continuous.
theorem constant_function_is_uniform(c: Real) {
    uniform(constant[Real, Real](c))
} by {
    constant_is_constant[Real, Real](c)
    constant_is_uniform(constant[Real, Real](c))
    uniform(constant[Real, Real](c))
}

/// A constant real function is continuous.
theorem constant_function_is_continuous(c: Real) {
    continuous(constant[Real, Real](c))
} by {
    constant_function_is_uniform(c)
    uniform_imp_continuous(constant[Real, Real](c))
    continuous(constant[Real, Real](c))
}

/// A constant real function is continuous at each point.
theorem constant_function_is_continuous_at(c: Real, x: Real) {
    continuous_at(constant[Real, Real](c), x)
} by {
    constant_function_is_continuous(c)
    continuous_imp_continuous_at(constant[Real, Real](c), x)
    continuous_at(constant[Real, Real](c), x)
}

/// Local continuity conditions compose.
theorem continuous_condition_compose(
    f: Real -> Real, g: Real -> Real, x: Real, eta: Real, delta: Real, eps: Real
) {
    continuous_condition(f, g(x), eta, eps) and continuous_condition(g, x, delta, eta)
    implies continuous_condition(compose(f, g), x, delta, eps)
} by {
    continuous_condition(f, g(x), eta, eps) = forall(z: Real) {
        z.is_close(g(x), eta) implies f(z).is_close(f(g(x)), eps)
    }
    continuous_condition(g, x, delta, eta) = forall(y: Real) {
        y.is_close(x, delta) implies g(y).is_close(g(x), eta)
    }
    forall(y: Real) {
        if y.is_close(x, delta) {
            g(y).is_close(g(x), eta)
            f(g(y)).is_close(f(g(x)), eps)
            compose(f, g, y) = f(g(y))
            compose(f, g, x) = f(g(x))
            compose(f, g, y).is_close(compose(f, g, x), eps)
        }
    }
}

/// Composition preserves continuity at a point.
theorem continuous_at_compose(f: Real -> Real, g: Real -> Real, x: Real) {
    continuous_at(g, x) and continuous_at(f, g(x)) implies continuous_at(compose(f, g), x)
} by {
    forall(eps: Real) {
        if eps.is_positive {
            continuous_at_delta(f, g(x), eps)
            let eta: Real satisfy {
                eta.is_positive and continuous_condition(f, g(x), eta, eps)
            }
            continuous_at_delta(g, x, eta)
            let delta: Real satisfy {
                delta.is_positive and continuous_condition(g, x, delta, eta)
            }
            continuous_condition_compose(f, g, x, eta, delta, eps)
            continuous_condition(compose(f, g), x, delta, eps)
            delta.is_positive and continuous_condition(compose(f, g), x, delta, eps)
            exists(delta2: Real) {
                delta2.is_positive and continuous_condition(compose(f, g), x, delta2, eps)
            }
        }
    }
    forall(eps2: Real) {
        eps2.is_positive implies exists(delta2: Real) {
            delta2.is_positive and continuous_condition(compose(f, g), x, delta2, eps2)
        }
    }
    continuous_at(compose(f, g), x) = forall(eps2: Real) {
        eps2.is_positive implies exists(delta2: Real) {
            delta2.is_positive and continuous_condition(compose(f, g), x, delta2, eps2)
        }
    }
    continuous_at_intro(compose(f, g), x)
    if not continuous_at(compose(f, g), x) {
        not forall(eps2: Real) {
            eps2.is_positive implies exists(delta2: Real) {
                delta2.is_positive and continuous_condition(compose(f, g), x, delta2, eps2)
            }
        }
        let bad_eps: Real satisfy {
            bad_eps.is_positive and forall(delta2: Real) {
                not (delta2.is_positive and continuous_condition(compose(f, g), x, delta2, bad_eps))
            }
        }
        exists(delta2: Real) {
            delta2.is_positive and continuous_condition(compose(f, g), x, delta2, bad_eps)
        }
        false
    }
    continuous_at(compose(f, g), x)
}

/// Composition preserves continuous real functions.
theorem continuous_compose(f: Real -> Real, g: Real -> Real) {
    continuous(f) and continuous(g) implies continuous(compose(f, g))
} by {
    continuous(f) = forall(y: Real) {
        continuous_at(f, y)
    }
    continuous(g) = forall(y: Real) {
        continuous_at(g, y)
    }
    forall(x: Real) {
        continuous_at(g, x)
        continuous_at(f, g(x))
        continuous_at_compose(f, g, x)
        continuous_at(compose(f, g), x)
    }
    continuous(compose(f, g)) = forall(x: Real) {
        continuous_at(compose(f, g), x)
    }
}
