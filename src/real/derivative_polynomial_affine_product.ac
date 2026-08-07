from data.basic.function_algebra import pointwise_add, pointwise_mul
from data.basic.functions import identity_fn
from real.continuity_cube import cube_real
from real.continuity_square import square_real
from real.derivative_affine_product import derivative_affine_mul,
    derivative_affine_square_mul, derivative_mul_affine,
    derivative_mul_affine_square, differentiable_affine_mul,
    differentiable_affine_square_mul, differentiable_mul_affine,
    differentiable_mul_affine_square
from real.derivative_basic import has_derivative_at, differentiable_at
from real.derivative_polynomial_chain import cube_real_differentiable_at,
    cube_real_has_derivative_at, square_real_differentiable_at,
    square_real_has_derivative_at
from real.real_base import Real

/// Multiplying the square function on the left by an affine factor follows the product rule.
theorem derivative_affine_mul_square_real(c: Real, b: Real, x0: Real) {
    has_derivative_at(
        pointwise_mul(
            pointwise_add(pointwise_mul(constant[Real, Real](c), identity_fn[Real]), constant[Real, Real](b)),
            square_real
        ),
        x0,
        pointwise_add(pointwise_mul(constant[Real, Real](c), identity_fn[Real]), constant[Real, Real](b), x0) *
        (x0 * Real.1 + x0 * Real.1) + square_real(x0) * c
    )
} by {
    let affine = pointwise_add(pointwise_mul(constant[Real, Real](c), identity_fn[Real]), constant[Real, Real](b))
    square_real_has_derivative_at(x0)
    derivative_affine_mul(c, b, square_real, x0, x0 * Real.1 + x0 * Real.1)
    has_derivative_at(pointwise_mul(affine, square_real), x0,
        affine(x0) * (x0 * Real.1 + x0 * Real.1) + square_real(x0) * c)
}

/// Multiplying the square function on the right by an affine factor follows the product rule.
theorem derivative_square_real_mul_affine(c: Real, b: Real, x0: Real) {
    has_derivative_at(
        pointwise_mul(
            square_real,
            pointwise_add(pointwise_mul(constant[Real, Real](c), identity_fn[Real]), constant[Real, Real](b))
        ),
        x0,
        square_real(x0) * c +
        pointwise_add(pointwise_mul(constant[Real, Real](c), identity_fn[Real]), constant[Real, Real](b), x0) *
        (x0 * Real.1 + x0 * Real.1)
    )
} by {
    let affine = pointwise_add(pointwise_mul(constant[Real, Real](c), identity_fn[Real]), constant[Real, Real](b))
    square_real_has_derivative_at(x0)
    derivative_mul_affine(square_real, c, b, x0, x0 * Real.1 + x0 * Real.1)
    has_derivative_at(pointwise_mul(square_real, affine), x0,
        square_real(x0) * c + affine(x0) * (x0 * Real.1 + x0 * Real.1))
}

/// Multiplying the cube function on the left by an affine factor follows the product rule.
theorem derivative_affine_mul_cube_real(c: Real, b: Real, x0: Real) {
    has_derivative_at(
        pointwise_mul(
            pointwise_add(pointwise_mul(constant[Real, Real](c), identity_fn[Real]), constant[Real, Real](b)),
            cube_real
        ),
        x0,
        pointwise_add(pointwise_mul(constant[Real, Real](c), identity_fn[Real]), constant[Real, Real](b), x0) *
        (square_real(x0) * Real.1 + x0 * (x0 * Real.1 + x0 * Real.1)) + cube_real(x0) * c
    )
} by {
    let affine = pointwise_add(pointwise_mul(constant[Real, Real](c), identity_fn[Real]), constant[Real, Real](b))
    cube_real_has_derivative_at(x0)
    derivative_affine_mul(c, b, cube_real, x0, square_real(x0) * Real.1 + x0 * (x0 * Real.1 + x0 * Real.1))
    has_derivative_at(pointwise_mul(affine, cube_real), x0,
        affine(x0) * (square_real(x0) * Real.1 + x0 * (x0 * Real.1 + x0 * Real.1)) + cube_real(x0) * c)
}

/// Multiplying the cube function on the right by an affine factor follows the product rule.
theorem derivative_cube_real_mul_affine(c: Real, b: Real, x0: Real) {
    has_derivative_at(
        pointwise_mul(
            cube_real,
            pointwise_add(pointwise_mul(constant[Real, Real](c), identity_fn[Real]), constant[Real, Real](b))
        ),
        x0,
        cube_real(x0) * c +
        pointwise_add(pointwise_mul(constant[Real, Real](c), identity_fn[Real]), constant[Real, Real](b), x0) *
        (square_real(x0) * Real.1 + x0 * (x0 * Real.1 + x0 * Real.1))
    )
} by {
    let affine = pointwise_add(pointwise_mul(constant[Real, Real](c), identity_fn[Real]), constant[Real, Real](b))
    cube_real_has_derivative_at(x0)
    derivative_mul_affine(cube_real, c, b, x0, square_real(x0) * Real.1 + x0 * (x0 * Real.1 + x0 * Real.1))
    has_derivative_at(pointwise_mul(cube_real, affine), x0,
        cube_real(x0) * c + affine(x0) * (square_real(x0) * Real.1 + x0 * (x0 * Real.1 + x0 * Real.1)))
}

/// Multiplying the square function on the left by an affine square follows the product rule.
theorem derivative_affine_square_mul_square_real(c: Real, b: Real, x0: Real) {
    has_derivative_at(
        pointwise_mul(
            pointwise_mul(
                pointwise_add(pointwise_mul(constant[Real, Real](c), identity_fn[Real]), constant[Real, Real](b)),
                pointwise_add(pointwise_mul(constant[Real, Real](c), identity_fn[Real]), constant[Real, Real](b))
            ),
            square_real
        ),
        x0,
        pointwise_mul(
            pointwise_add(pointwise_mul(constant[Real, Real](c), identity_fn[Real]), constant[Real, Real](b)),
            pointwise_add(pointwise_mul(constant[Real, Real](c), identity_fn[Real]), constant[Real, Real](b)),
            x0
        ) * (x0 * Real.1 + x0 * Real.1) +
        square_real(x0) * (
            pointwise_add(pointwise_mul(constant[Real, Real](c), identity_fn[Real]), constant[Real, Real](b), x0) * c +
            pointwise_add(pointwise_mul(constant[Real, Real](c), identity_fn[Real]), constant[Real, Real](b), x0) * c
        )
    )
} by {
    square_real_has_derivative_at(x0)
    derivative_affine_square_mul(c, b, square_real, x0, x0 * Real.1 + x0 * Real.1)
}

/// Multiplying the square function on the right by an affine square follows the product rule.
theorem derivative_square_real_mul_affine_square(c: Real, b: Real, x0: Real) {
    has_derivative_at(
        pointwise_mul(
            square_real,
            pointwise_mul(
                pointwise_add(pointwise_mul(constant[Real, Real](c), identity_fn[Real]), constant[Real, Real](b)),
                pointwise_add(pointwise_mul(constant[Real, Real](c), identity_fn[Real]), constant[Real, Real](b))
            )
        ),
        x0,
        square_real(x0) * (
            pointwise_add(pointwise_mul(constant[Real, Real](c), identity_fn[Real]), constant[Real, Real](b), x0) * c +
            pointwise_add(pointwise_mul(constant[Real, Real](c), identity_fn[Real]), constant[Real, Real](b), x0) * c
        ) + pointwise_mul(
            pointwise_add(pointwise_mul(constant[Real, Real](c), identity_fn[Real]), constant[Real, Real](b)),
            pointwise_add(pointwise_mul(constant[Real, Real](c), identity_fn[Real]), constant[Real, Real](b)),
            x0
        ) * (x0 * Real.1 + x0 * Real.1)
    )
} by {
    square_real_has_derivative_at(x0)
    derivative_mul_affine_square(square_real, c, b, x0, x0 * Real.1 + x0 * Real.1)
}

/// Multiplying the cube function on the left by an affine square follows the product rule.
theorem derivative_affine_square_mul_cube_real(c: Real, b: Real, x0: Real) {
    has_derivative_at(
        pointwise_mul(
            pointwise_mul(
                pointwise_add(pointwise_mul(constant[Real, Real](c), identity_fn[Real]), constant[Real, Real](b)),
                pointwise_add(pointwise_mul(constant[Real, Real](c), identity_fn[Real]), constant[Real, Real](b))
            ),
            cube_real
        ),
        x0,
        pointwise_mul(
            pointwise_add(pointwise_mul(constant[Real, Real](c), identity_fn[Real]), constant[Real, Real](b)),
            pointwise_add(pointwise_mul(constant[Real, Real](c), identity_fn[Real]), constant[Real, Real](b)),
            x0
        ) * (square_real(x0) * Real.1 + x0 * (x0 * Real.1 + x0 * Real.1)) +
        cube_real(x0) * (
            pointwise_add(pointwise_mul(constant[Real, Real](c), identity_fn[Real]), constant[Real, Real](b), x0) * c +
            pointwise_add(pointwise_mul(constant[Real, Real](c), identity_fn[Real]), constant[Real, Real](b), x0) * c
        )
    )
} by {
    cube_real_has_derivative_at(x0)
    derivative_affine_square_mul(c, b, cube_real, x0, square_real(x0) * Real.1 + x0 * (x0 * Real.1 + x0 * Real.1))
}

/// Multiplying the cube function on the right by an affine square follows the product rule.
theorem derivative_cube_real_mul_affine_square(c: Real, b: Real, x0: Real) {
    has_derivative_at(
        pointwise_mul(
            cube_real,
            pointwise_mul(
                pointwise_add(pointwise_mul(constant[Real, Real](c), identity_fn[Real]), constant[Real, Real](b)),
                pointwise_add(pointwise_mul(constant[Real, Real](c), identity_fn[Real]), constant[Real, Real](b))
            )
        ),
        x0,
        cube_real(x0) * (
            pointwise_add(pointwise_mul(constant[Real, Real](c), identity_fn[Real]), constant[Real, Real](b), x0) * c +
            pointwise_add(pointwise_mul(constant[Real, Real](c), identity_fn[Real]), constant[Real, Real](b), x0) * c
        ) + pointwise_mul(
            pointwise_add(pointwise_mul(constant[Real, Real](c), identity_fn[Real]), constant[Real, Real](b)),
            pointwise_add(pointwise_mul(constant[Real, Real](c), identity_fn[Real]), constant[Real, Real](b)),
            x0
        ) * (square_real(x0) * Real.1 + x0 * (x0 * Real.1 + x0 * Real.1))
    )
} by {
    cube_real_has_derivative_at(x0)
    derivative_mul_affine_square(cube_real, c, b, x0, square_real(x0) * Real.1 + x0 * (x0 * Real.1 + x0 * Real.1))
}

/// Differentiability is preserved by multiplying the square function on the left by an affine factor.
theorem differentiable_affine_mul_square_real(c: Real, b: Real, x0: Real) {
    differentiable_at(
        pointwise_mul(
            pointwise_add(pointwise_mul(constant[Real, Real](c), identity_fn[Real]), constant[Real, Real](b)),
            square_real
        ),
        x0
    )
} by {
    square_real_differentiable_at(x0)
    differentiable_affine_mul(c, b, square_real, x0)
    differentiable_at(pointwise_mul(pointwise_add(pointwise_mul(constant[Real, Real](c), identity_fn[Real]), constant[Real, Real](b)), square_real), x0)
}

/// Differentiability is preserved by multiplying the square function on the right by an affine factor.
theorem differentiable_square_real_mul_affine(c: Real, b: Real, x0: Real) {
    differentiable_at(
        pointwise_mul(square_real, pointwise_add(pointwise_mul(constant[Real, Real](c), identity_fn[Real]), constant[Real, Real](b))),
        x0
    )
} by {
    square_real_differentiable_at(x0)
    differentiable_mul_affine(square_real, c, b, x0)
    differentiable_at(pointwise_mul(square_real, pointwise_add(pointwise_mul(constant[Real, Real](c), identity_fn[Real]), constant[Real, Real](b))), x0)
}

/// Differentiability is preserved by multiplying the cube function on the left by an affine factor.
theorem differentiable_affine_mul_cube_real(c: Real, b: Real, x0: Real) {
    differentiable_at(
        pointwise_mul(pointwise_add(pointwise_mul(constant[Real, Real](c), identity_fn[Real]), constant[Real, Real](b)), cube_real),
        x0
    )
} by {
    cube_real_differentiable_at(x0)
    differentiable_affine_mul(c, b, cube_real, x0)
    differentiable_at(pointwise_mul(pointwise_add(pointwise_mul(constant[Real, Real](c), identity_fn[Real]), constant[Real, Real](b)), cube_real), x0)
}

/// Differentiability is preserved by multiplying the cube function on the right by an affine factor.
theorem differentiable_cube_real_mul_affine(c: Real, b: Real, x0: Real) {
    differentiable_at(
        pointwise_mul(cube_real, pointwise_add(pointwise_mul(constant[Real, Real](c), identity_fn[Real]), constant[Real, Real](b))),
        x0
    )
} by {
    cube_real_differentiable_at(x0)
    differentiable_mul_affine(cube_real, c, b, x0)
    differentiable_at(pointwise_mul(cube_real, pointwise_add(pointwise_mul(constant[Real, Real](c), identity_fn[Real]), constant[Real, Real](b))), x0)
}

/// Differentiability is preserved by multiplying the square function on the left by an affine square.
theorem differentiable_affine_square_mul_square_real(c: Real, b: Real, x0: Real) {
    differentiable_at(
        pointwise_mul(
            pointwise_mul(
                pointwise_add(pointwise_mul(constant[Real, Real](c), identity_fn[Real]), constant[Real, Real](b)),
                pointwise_add(pointwise_mul(constant[Real, Real](c), identity_fn[Real]), constant[Real, Real](b))
            ),
            square_real
        ),
        x0
    )
} by {
    square_real_differentiable_at(x0)
    differentiable_affine_square_mul(c, b, square_real, x0)
    differentiable_at(pointwise_mul(pointwise_mul(pointwise_add(pointwise_mul(constant[Real, Real](c), identity_fn[Real]), constant[Real, Real](b)), pointwise_add(pointwise_mul(constant[Real, Real](c), identity_fn[Real]), constant[Real, Real](b))), square_real), x0)
}

/// Differentiability is preserved by multiplying the square function on the right by an affine square.
theorem differentiable_square_real_mul_affine_square(c: Real, b: Real, x0: Real) {
    differentiable_at(
        pointwise_mul(
            square_real,
            pointwise_mul(
                pointwise_add(pointwise_mul(constant[Real, Real](c), identity_fn[Real]), constant[Real, Real](b)),
                pointwise_add(pointwise_mul(constant[Real, Real](c), identity_fn[Real]), constant[Real, Real](b))
            )
        ),
        x0
    )
} by {
    square_real_differentiable_at(x0)
    differentiable_mul_affine_square(square_real, c, b, x0)
    differentiable_at(pointwise_mul(square_real, pointwise_mul(pointwise_add(pointwise_mul(constant[Real, Real](c), identity_fn[Real]), constant[Real, Real](b)), pointwise_add(pointwise_mul(constant[Real, Real](c), identity_fn[Real]), constant[Real, Real](b)))), x0)
}

/// Differentiability is preserved by multiplying the cube function on the left by an affine square.
theorem differentiable_affine_square_mul_cube_real(c: Real, b: Real, x0: Real) {
    differentiable_at(
        pointwise_mul(
            pointwise_mul(
                pointwise_add(pointwise_mul(constant[Real, Real](c), identity_fn[Real]), constant[Real, Real](b)),
                pointwise_add(pointwise_mul(constant[Real, Real](c), identity_fn[Real]), constant[Real, Real](b))
            ),
            cube_real
        ),
        x0
    )
} by {
    cube_real_differentiable_at(x0)
    differentiable_affine_square_mul(c, b, cube_real, x0)
    differentiable_at(pointwise_mul(pointwise_mul(pointwise_add(pointwise_mul(constant[Real, Real](c), identity_fn[Real]), constant[Real, Real](b)), pointwise_add(pointwise_mul(constant[Real, Real](c), identity_fn[Real]), constant[Real, Real](b))), cube_real), x0)
}

/// Differentiability is preserved by multiplying the cube function on the right by an affine square.
theorem differentiable_cube_real_mul_affine_square(c: Real, b: Real, x0: Real) {
    differentiable_at(
        pointwise_mul(
            cube_real,
            pointwise_mul(
                pointwise_add(pointwise_mul(constant[Real, Real](c), identity_fn[Real]), constant[Real, Real](b)),
                pointwise_add(pointwise_mul(constant[Real, Real](c), identity_fn[Real]), constant[Real, Real](b))
            )
        ),
        x0
    )
} by {
    cube_real_differentiable_at(x0)
    differentiable_mul_affine_square(cube_real, c, b, x0)
    differentiable_at(pointwise_mul(cube_real, pointwise_mul(pointwise_add(pointwise_mul(constant[Real, Real](c), identity_fn[Real]), constant[Real, Real](b)), pointwise_add(pointwise_mul(constant[Real, Real](c), identity_fn[Real]), constant[Real, Real](b)))), x0)
}
