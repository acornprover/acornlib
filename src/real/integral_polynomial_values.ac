/// Integrals of polynomial functions over closed intervals, computed with
/// the fundamental theorem of calculus.
///
/// Each integrand f is paired with an explicit polynomial antiderivative g
/// built from the pointwise combinators (const_mul_left, pointwise_add,
/// pointwise_mul) of the calculus API.  The derivative and continuity of g are
/// transported from the API rules, integrability of f follows from the
/// Lipschitz criterion fn_integrable_gen, and the value is then
/// integral(f, a, b) = g(b) - g(a) by ftc2_general.  The results here feed the
/// continuous probability distributions in real.uniform_distribution.

from nat import Nat
from order import lt_trans, lt_imp_ne, lte_trans, lt_imp_lte
from data.basic.functions import identity_fn, compose, function_extensionality, function_eq_transport_predicate_rev
from real.continuity_affine import affine_real, continuous_affine_real
from real.calculus_api import derivative_fn_affine_identity
from data.basic.function_algebra import pointwise_mul, pointwise_add
from data.basic.logic import eq_true_intro
from real.real_field import Real
from real.real_base import add_comm, add_assoc, neg_neg, lt_add_right, sub_cancels, lte_abs, abs_neg, abs_gte_zero, neg_distrib
from real.real_ring import mul_sub_distrib_left
from real.integral import integral, is_integrable, interval_contains, interval_contains_left, interval_contains_right, neg_lte_flip
from real.calculus_api import is_derivative_fn, is_derivative_fn_at, is_derivative_fn_iff, derivative_fn_identity, derivative_fn_square, derivative_fn_const_mul, derivative_fn_mul, derivative_fn_add, derivative_fn_compose
from real.derivative_basic import has_derivative_at
from algebra.ring.ring import mul_neg_neg
from real.continuity_base import continuous
from real.continuity_const_mul import const_mul_left, continuous_const_mul_left, pointwise_mul_constant_left_eq
from real.continuity_square import square_real, continuous_square_real, square_real_eq_pointwise_mul_identity
from real.continuity_cube import cube_real, cube_real_eq_pointwise_mul_square_identity, continuous_cube_real
from real.exp import two, three, two_positive, two_nonzero, mul_one_over, mul_frac_assoc, one_half_plus_one_half
from real.harmonic import from_nat_suc_pos_real
from real.integral_exp import ftc2_general
from real.integral_trig import fn_integrable_gen
from real.derivative_continuity import div_mul_cancel_denominator
from real.real_ring import mul_assoc, mul_pos_pos, real_mul_comm, mul_abs
from ordered_field import mul_le_mul_of_nonneg_right
from algebra.add_ordered_group import add_le_add_right, add_le_add
from real.real_ring import square_nonneg
from real.derivative_trig import abs_of_nonneg
from real.continuity_composition import constant_function_is_continuous, continuous_compose
from real.continuity_sequences import identity_function_is_continuous
from real.continuity_pointwise_mul import continuous_pointwise_mul
from real.continuity_pointwise import continuous_pointwise_add
from real.real_field import mul_left_cancel
from ordered_field import zero_is_smaller_than_one
from real.integral_polynomial import interval_contains_mono, integral_eq_of_pointwise_eq_on, is_integrable_eq_of_pointwise_eq_on

numerals Real
numerals Nat

/// One third, as a real number.
let one_third = Real.1 / three

/// Three times one third is one.
lemma three_pos {
    Real.0 < three
} by {
    two_positive
    two > Real.0
    lt_add_right(Real.0, two, Real.1)
    Real.0 + Real.1 < two + Real.1
    Real.0 + Real.1 = Real.1
    two + Real.1 = three
    Real.1 < three
    zero_is_smaller_than_one[Real]
    Real.0 < Real.1
    lt_trans(Real.0, Real.1, three)
    Real.0 < three
}

/// Three is nonzero.
lemma three_ne_zero {
    three != Real.0
} by {
    three_pos
    Real.0 < three
    lt_imp_ne(Real.0, three)
    Real.0 != three
    three != Real.0
}

/// Three times one third is one.
lemma three_mul_one_third {
    three * one_third = Real.1
} by {
    three_ne_zero
    three != Real.0
    div_mul_cancel_denominator(Real.1, three)
    (Real.1 / three) * three = Real.1
    real_mul_comm(Real.1 / three, three)
    three * (Real.1 / three) = Real.1
    one_third = Real.1 / three
    three * one_third = Real.1
}

/// One half times two is one.
lemma one_half_mul_two {
    Real.one_half * two = Real.1
} by {
    one_half_plus_one_half
    Real.one_half + Real.one_half = Real.1
    two = Real.1 + Real.1
    Real.one_half * two = Real.one_half * (Real.1 + Real.1)
    Real.one_half * (Real.1 + Real.1) = Real.one_half + Real.one_half
    Real.one_half * two = Real.1
}

/// A global derivative function is unchanged when the derivative function is
/// replaced by a pointwise equal one.
lemma is_derivative_fn_pointwise_eq(f: Real -> Real, df: Real -> Real, dg: Real -> Real) {
    is_derivative_fn(f, df) and (forall(x: Real) { df(x) = dg(x) })
    implies is_derivative_fn(f, dg)
} by {
    if is_derivative_fn(f, df) and (forall(x: Real) { df(x) = dg(x) }) {
        forall(x: Real) {
            is_derivative_fn_at(f, df, x)
            has_derivative_at(f, x, df(x))
            df(x) = dg(x)
            has_derivative_at(f, x, dg(x))
        }
        is_derivative_fn_iff(f, dg)
        is_derivative_fn(f, dg) = forall(y: Real) {
            has_derivative_at(f, y, dg(y))
        }
        is_derivative_fn(f, dg)
    }
}

/// A global derivative function transports along pointwise equality of the
/// underlying function.
lemma is_derivative_fn_fn_eq(f: Real -> Real, g: Real -> Real, df: Real -> Real) {
    is_derivative_fn(f, df) and (forall(x: Real) { f(x) = g(x) })
    implies is_derivative_fn(g, df)
} by {
    if is_derivative_fn(f, df) and (forall(x: Real) { f(x) = g(x) }) {
        function_extensionality(f, g)
        f = g
        define derivative_pred(h: Real -> Real) -> Bool {
            is_derivative_fn(h, df)
        }
        derivative_pred(f)
        function_eq_transport_predicate_rev(derivative_pred, g, f)
        is_derivative_fn(g, df)
    }
}

/// A function with a global derivative has the transported derivative when
/// both the function and the derivative change pointwise.
lemma is_derivative_fn_both_eq(f: Real -> Real, g: Real -> Real, df: Real -> Real, dg: Real -> Real) {
    is_derivative_fn(f, df) and (forall(x: Real) { f(x) = g(x) }) and (forall(x: Real) { df(x) = dg(x) })
    implies is_derivative_fn(g, dg)
} by {
    if is_derivative_fn(f, df) and (forall(x: Real) { f(x) = g(x) }) and (forall(x: Real) { df(x) = dg(x) }) {
        is_derivative_fn_fn_eq(f, g, df)
        is_derivative_fn(g, df)
        is_derivative_fn_pointwise_eq(g, df, dg)
        is_derivative_fn(g, dg)
    }
}

/// The function x -> x^2 / 2 is continuous.
lemma half_square_continuous {
    continuous(const_mul_left(Real.one_half, square_real))
} by {
    continuous_square_real
    continuous(square_real)
    continuous_const_mul_left(Real.one_half, square_real)
    continuous(const_mul_left(Real.one_half, square_real))
}

/// The derivative of x -> x^2 / 2 is the identity.
lemma half_square_is_derivative_fn {
    is_derivative_fn(const_mul_left(Real.one_half, square_real), identity_fn[Real])
} by {
    derivative_fn_identity
    is_derivative_fn(identity_fn[Real], constant[Real, Real](Real.1))
    derivative_fn_square(identity_fn[Real], constant[Real, Real](Real.1))
    is_derivative_fn(pointwise_mul(identity_fn[Real], identity_fn[Real]),
        pointwise_add(pointwise_mul(identity_fn[Real], constant[Real, Real](Real.1)),
            pointwise_mul(identity_fn[Real], constant[Real, Real](Real.1))))
    square_real_eq_pointwise_mul_identity
    square_real = pointwise_mul(identity_fn[Real], identity_fn[Real])
    define square_derivative_pred(h: Real -> Real) -> Bool {
        is_derivative_fn(h, pointwise_add(pointwise_mul(identity_fn[Real], constant[Real, Real](Real.1)),
            pointwise_mul(identity_fn[Real], constant[Real, Real](Real.1))))
    }
    square_derivative_pred(pointwise_mul(identity_fn[Real], identity_fn[Real]))
    function_eq_transport_predicate_rev(square_derivative_pred, square_real,
        pointwise_mul(identity_fn[Real], identity_fn[Real]))
    is_derivative_fn(square_real,
        pointwise_add(pointwise_mul(identity_fn[Real], constant[Real, Real](Real.1)),
            pointwise_mul(identity_fn[Real], constant[Real, Real](Real.1))))
    derivative_fn_const_mul(Real.one_half, square_real,
        pointwise_add(pointwise_mul(identity_fn[Real], constant[Real, Real](Real.1)),
            pointwise_mul(identity_fn[Real], constant[Real, Real](Real.1))))
    is_derivative_fn(pointwise_mul(constant[Real, Real](Real.one_half), square_real),
        pointwise_mul(constant[Real, Real](Real.one_half),
            pointwise_add(pointwise_mul(identity_fn[Real], constant[Real, Real](Real.1)),
                pointwise_mul(identity_fn[Real], constant[Real, Real](Real.1)))))
    pointwise_mul_constant_left_eq(Real.one_half, square_real)
    pointwise_mul(constant[Real, Real](Real.one_half), square_real) = const_mul_left(Real.one_half, square_real)
    define half_square_pred(h: Real -> Real) -> Bool {
        is_derivative_fn(h, pointwise_mul(constant[Real, Real](Real.one_half),
            pointwise_add(pointwise_mul(identity_fn[Real], constant[Real, Real](Real.1)),
                pointwise_mul(identity_fn[Real], constant[Real, Real](Real.1)))))
    }
    half_square_pred(pointwise_mul(constant[Real, Real](Real.one_half), square_real))
    function_eq_transport_predicate_rev(half_square_pred, const_mul_left(Real.one_half, square_real),
        pointwise_mul(constant[Real, Real](Real.one_half), square_real))
    is_derivative_fn(const_mul_left(Real.one_half, square_real),
        pointwise_mul(constant[Real, Real](Real.one_half),
            pointwise_add(pointwise_mul(identity_fn[Real], constant[Real, Real](Real.1)),
                pointwise_mul(identity_fn[Real], constant[Real, Real](Real.1)))))
    forall(x: Real) {
        constant[Real, Real](Real.one_half, x) = Real.one_half
        pointwise_mul(constant[Real, Real](Real.one_half), pointwise_add(pointwise_mul(identity_fn[Real], constant[Real, Real](Real.1)), pointwise_mul(identity_fn[Real], constant[Real, Real](Real.1))), x) = Real.one_half * pointwise_add(pointwise_mul(identity_fn[Real], constant[Real, Real](Real.1)), pointwise_mul(identity_fn[Real], constant[Real, Real](Real.1)), x)
        pointwise_add(pointwise_mul(identity_fn[Real], constant[Real, Real](Real.1)), pointwise_mul(identity_fn[Real], constant[Real, Real](Real.1)), x) = pointwise_mul(identity_fn[Real], constant[Real, Real](Real.1), x) + pointwise_mul(identity_fn[Real], constant[Real, Real](Real.1), x)
        pointwise_mul(identity_fn[Real], constant[Real, Real](Real.1), x) = identity_fn[Real](x) * Real.1
        pointwise_mul(constant[Real, Real](Real.one_half), pointwise_add(pointwise_mul(identity_fn[Real], constant[Real, Real](Real.1)), pointwise_mul(identity_fn[Real], constant[Real, Real](Real.1))), x) = Real.one_half * (identity_fn[Real](x) * Real.1 + identity_fn[Real](x) * Real.1)
        identity_fn[Real](x) = x
        identity_fn[Real](x) * Real.1 = x
        identity_fn[Real](x) * Real.1 + identity_fn[Real](x) * Real.1 = x + x
        x + x = two * x
        Real.one_half * (two * x) = (Real.one_half * two) * x
        one_half_mul_two
        Real.one_half * two = Real.1
        (Real.one_half * two) * x = Real.1 * x
        Real.1 * x = x
        Real.one_half * (two * x) = x
        pointwise_mul(constant[Real, Real](Real.one_half),
            pointwise_add(pointwise_mul(identity_fn[Real], constant[Real, Real](Real.1)),
                pointwise_mul(identity_fn[Real], constant[Real, Real](Real.1))), x) = x
        identity_fn[Real](x) = x
        pointwise_mul(constant[Real, Real](Real.one_half),
            pointwise_add(pointwise_mul(identity_fn[Real], constant[Real, Real](Real.1)),
                pointwise_mul(identity_fn[Real], constant[Real, Real](Real.1))), x) = identity_fn[Real](x)
    }
    function_extensionality(pointwise_mul(constant[Real, Real](Real.one_half),
        pointwise_add(pointwise_mul(identity_fn[Real], constant[Real, Real](Real.1)),
            pointwise_mul(identity_fn[Real], constant[Real, Real](Real.1)))),
        identity_fn[Real])
    pointwise_mul(constant[Real, Real](Real.one_half),
        pointwise_add(pointwise_mul(identity_fn[Real], constant[Real, Real](Real.1)),
            pointwise_mul(identity_fn[Real], constant[Real, Real](Real.1)))) = identity_fn[Real]
    is_derivative_fn_pointwise_eq(const_mul_left(Real.one_half, square_real),
        pointwise_mul(constant[Real, Real](Real.one_half),
            pointwise_add(pointwise_mul(identity_fn[Real], constant[Real, Real](Real.1)),
                pointwise_mul(identity_fn[Real], constant[Real, Real](Real.1)))),
        identity_fn[Real])
    is_derivative_fn(const_mul_left(Real.one_half, square_real), identity_fn[Real])
}

/// On [a, b] the identity is bounded below by a.
lemma identity_lower_bound_on(a: Real, b: Real) {
    a <= b implies forall(t: Real) { interval_contains(a, b, t) implies a <= identity_fn[Real](t) }
} by {
    if a <= b {
        forall(t: Real) {
            if interval_contains(a, b, t) {
                interval_contains_left(a, b, t)
                a <= t
                identity_fn[Real](t) = t
                a <= identity_fn[Real](t)
            }
        }
    }
}

/// On [a, b] the identity is bounded above by b.
lemma identity_upper_bound_on(a: Real, b: Real) {
    a <= b implies forall(t: Real) { interval_contains(a, b, t) implies identity_fn[Real](t) <= b }
} by {
    if a <= b {
        forall(t: Real) {
            if interval_contains(a, b, t) {
                interval_contains_right(a, b, t)
                t <= b
                identity_fn[Real](t) = t
                identity_fn[Real](t) <= b
            }
        }
    }
}

/// The identity is one-Lipschitz on every interval.
lemma identity_lipschitz(a: Real, b: Real) {
    forall(u: Real, v: Real) {
        interval_contains(a, b, u) and interval_contains(a, b, v)
        implies (identity_fn[Real](u) - identity_fn[Real](v)).abs <= Real.1 * (u - v).abs
    }
} by {
    forall(u: Real, v: Real) {
        if interval_contains(a, b, u) and interval_contains(a, b, v) {
            identity_fn[Real](u) = u
            identity_fn[Real](v) = v
            identity_fn[Real](u) - identity_fn[Real](v) = u - v
            (identity_fn[Real](u) - identity_fn[Real](v)).abs = (u - v).abs
            Real.1 * (u - v).abs = (u - v).abs
            (identity_fn[Real](u) - identity_fn[Real](v)).abs <= Real.1 * (u - v).abs
        }
    }
}

/// The identity is integrable on every closed interval.
theorem identity_integrable(a: Real, b: Real) {
    a <= b implies is_integrable(identity_fn[Real], a, b)
} by {
    if a <= b {
        half_square_continuous
        continuous(const_mul_left(Real.one_half, square_real))
        half_square_is_derivative_fn
        is_derivative_fn(const_mul_left(Real.one_half, square_real), identity_fn[Real])
        identity_lipschitz(a, b)
        identity_lower_bound_on(a, b)
        identity_upper_bound_on(a, b)
        Real.0 <= Real.1
        eq_true_intro(continuous(const_mul_left(Real.one_half, square_real)))
        (continuous(const_mul_left(Real.one_half, square_real))) = true
        eq_true_intro(is_derivative_fn(const_mul_left(Real.one_half, square_real), identity_fn[Real]))
        (is_derivative_fn(const_mul_left(Real.one_half, square_real), identity_fn[Real])) = true
        eq_true_intro(forall(u: Real, v: Real) {
            interval_contains(a, b, u) and interval_contains(a, b, v)
            implies (identity_fn[Real](u) - identity_fn[Real](v)).abs <= Real.1 * (u - v).abs
        })
        (forall(u: Real, v: Real) {
            interval_contains(a, b, u) and interval_contains(a, b, v)
            implies (identity_fn[Real](u) - identity_fn[Real](v)).abs <= Real.1 * (u - v).abs
        }) = true
        eq_true_intro(forall(t: Real) { interval_contains(a, b, t) implies a <= identity_fn[Real](t) })
        (forall(t: Real) { interval_contains(a, b, t) implies a <= identity_fn[Real](t) }) = true
        eq_true_intro(forall(t: Real) { interval_contains(a, b, t) implies identity_fn[Real](t) <= b })
        (forall(t: Real) { interval_contains(a, b, t) implies identity_fn[Real](t) <= b }) = true
        a <= b and Real.0 <= Real.1 and
            continuous(const_mul_left(Real.one_half, square_real)) and
            is_derivative_fn(const_mul_left(Real.one_half, square_real), identity_fn[Real]) and
            (forall(u: Real, v: Real) {
                interval_contains(a, b, u) and interval_contains(a, b, v)
                implies (identity_fn[Real](u) - identity_fn[Real](v)).abs <= Real.1 * (u - v).abs
            }) and
            (forall(t: Real) { interval_contains(a, b, t) implies a <= identity_fn[Real](t) }) and
            (forall(t: Real) { interval_contains(a, b, t) implies identity_fn[Real](t) <= b })
        fn_integrable_gen(identity_fn[Real], const_mul_left(Real.one_half, square_real), a, b, Real.1, a, b)
        is_integrable(identity_fn[Real], a, b)
    }
}

/// The identity is integrable on [a, b] and its integral is (b^2 - a^2) / 2.
theorem integral_identity(a: Real, b: Real) {
    a <= b implies (is_integrable(identity_fn[Real], a, b) and integral(identity_fn[Real], a, b) = (b * b - a * a) * Real.one_half)
} by {
    if a <= b {
        identity_integrable(a, b)
        is_integrable(identity_fn[Real], a, b)
        half_square_continuous
        continuous(const_mul_left(Real.one_half, square_real))
        half_square_is_derivative_fn
        is_derivative_fn(const_mul_left(Real.one_half, square_real), identity_fn[Real])
        identity_lower_bound_on(a, b)
        identity_upper_bound_on(a, b)
        eq_true_intro(continuous(const_mul_left(Real.one_half, square_real)))
        (continuous(const_mul_left(Real.one_half, square_real))) = true
        eq_true_intro(is_derivative_fn(const_mul_left(Real.one_half, square_real), identity_fn[Real]))
        (is_derivative_fn(const_mul_left(Real.one_half, square_real), identity_fn[Real])) = true
        eq_true_intro(forall(t: Real) { interval_contains(a, b, t) implies a <= identity_fn[Real](t) })
        (forall(t: Real) { interval_contains(a, b, t) implies a <= identity_fn[Real](t) }) = true
        eq_true_intro(forall(t: Real) { interval_contains(a, b, t) implies identity_fn[Real](t) <= b })
        (forall(t: Real) { interval_contains(a, b, t) implies identity_fn[Real](t) <= b }) = true
        a <= b and is_integrable(identity_fn[Real], a, b) and
            continuous(const_mul_left(Real.one_half, square_real)) and
            is_derivative_fn(const_mul_left(Real.one_half, square_real), identity_fn[Real]) and
            (forall(t: Real) { interval_contains(a, b, t) implies a <= identity_fn[Real](t) }) and
            (forall(t: Real) { interval_contains(a, b, t) implies identity_fn[Real](t) <= b })
        ftc2_general(identity_fn[Real], const_mul_left(Real.one_half, square_real), a, b, a, b)
        integral(identity_fn[Real], a, b) =
            const_mul_left(Real.one_half, square_real, b) - const_mul_left(Real.one_half, square_real, a)
        const_mul_left(Real.one_half, square_real, b) = Real.one_half * square_real(b)
        const_mul_left(Real.one_half, square_real, a) = Real.one_half * square_real(a)
        square_real(b) = b * b
        square_real(a) = a * a
        const_mul_left(Real.one_half, square_real, b) = Real.one_half * (b * b)
        const_mul_left(Real.one_half, square_real, a) = Real.one_half * (a * a)
        integral(identity_fn[Real], a, b) = Real.one_half * (b * b) - Real.one_half * (a * a)
        Real.one_half * (b * b) - Real.one_half * (a * a) = (b * b - a * a) * Real.one_half
        integral(identity_fn[Real], a, b) = (b * b - a * a) * Real.one_half
        is_integrable(identity_fn[Real], a, b) and
            integral(identity_fn[Real], a, b) = (b * b - a * a) * Real.one_half
        is_integrable(identity_fn[Real], a, b) and integral(identity_fn[Real], a, b) = (b * b - a * a) * Real.one_half
    }
}

// ---------------------------------------------------------------------------
// The integral of the centered quadratic (x - m)^2
// ---------------------------------------------------------------------------
//
// The variance of the uniform law on [0, 1] is the integral of the centered
// quadratic (x - 1/2)^2 over [0, 1].  The integrand is the pointwise
// composition of the square with the shift x -> x - 1/2, and the
// antiderivative is one third of the cube of the same shift; both derivative
// facts follow from the chain rule of the calculus API.

/// The affine shift x -> x - 1/2.
define unit_shift(x: Real) -> Real {
    x - Real.one_half
}

/// The centered quadratic (x - 1/2)^2.
define centered_square_unit(x: Real) -> Real {
    (x - Real.one_half) * (x - Real.one_half)
}

/// The product-rule derivative function of the square, x -> 2x.
define square_dx(x: Real) -> Real {
    x * Real.1 + x * Real.1
}

/// The product-rule derivative function of the cube, x -> 3x^2.
define cube_dx(x: Real) -> Real {
    square_real(x) * Real.1 + x * (x * Real.1 + x * Real.1)
}

/// The square-derivative combinator agrees with the named derivative function.
lemma square_dx_pointwise_eq(x: Real) {
    pointwise_add(pointwise_mul(identity_fn[Real], constant[Real, Real](Real.1)),
        pointwise_mul(identity_fn[Real], constant[Real, Real](Real.1)), x) = square_dx(x)
} by {
    pointwise_mul(identity_fn[Real], constant[Real, Real](Real.1), x) = identity_fn[Real](x) * Real.1
    identity_fn[Real](x) = x
    pointwise_mul(identity_fn[Real], constant[Real, Real](Real.1), x) = x * Real.1
    x * Real.1 = x
    pointwise_mul(identity_fn[Real], constant[Real, Real](Real.1), x) = x
    pointwise_add(pointwise_mul(identity_fn[Real], constant[Real, Real](Real.1)), pointwise_mul(identity_fn[Real], constant[Real, Real](Real.1)), x) = pointwise_mul(identity_fn[Real], constant[Real, Real](Real.1), x) + pointwise_mul(identity_fn[Real], constant[Real, Real](Real.1), x)
    pointwise_mul(identity_fn[Real], constant[Real, Real](Real.1), x) + pointwise_mul(identity_fn[Real], constant[Real, Real](Real.1), x) = x + x
    pointwise_add(pointwise_mul(identity_fn[Real], constant[Real, Real](Real.1)),
        pointwise_mul(identity_fn[Real], constant[Real, Real](Real.1)), x) = x + x
    square_dx(x) = x * Real.1 + x * Real.1
    x * Real.1 = x
    square_dx(x) = x + x
    pointwise_add(pointwise_mul(identity_fn[Real], constant[Real, Real](Real.1)),
        pointwise_mul(identity_fn[Real], constant[Real, Real](Real.1)), x) = square_dx(x)
}

/// The square has the product-rule derivative combinator.
lemma square_has_derivative_combinator {
    is_derivative_fn(square_real,
        pointwise_add(pointwise_mul(identity_fn[Real], constant[Real, Real](Real.1)),
            pointwise_mul(identity_fn[Real], constant[Real, Real](Real.1))))
} by {
    derivative_fn_identity
    is_derivative_fn(identity_fn[Real], constant[Real, Real](Real.1))
    derivative_fn_square(identity_fn[Real], constant[Real, Real](Real.1))
    is_derivative_fn(pointwise_mul(identity_fn[Real], identity_fn[Real]),
        pointwise_add(pointwise_mul(identity_fn[Real], constant[Real, Real](Real.1)),
            pointwise_mul(identity_fn[Real], constant[Real, Real](Real.1))))
    square_real_eq_pointwise_mul_identity
    square_real = pointwise_mul(identity_fn[Real], identity_fn[Real])
    define square_derivative_pred(h: Real -> Real) -> Bool {
        is_derivative_fn(h,
            pointwise_add(pointwise_mul(identity_fn[Real], constant[Real, Real](Real.1)),
                pointwise_mul(identity_fn[Real], constant[Real, Real](Real.1))))
    }
    square_derivative_pred(pointwise_mul(identity_fn[Real], identity_fn[Real]))
    function_eq_transport_predicate_rev(square_derivative_pred, square_real,
        pointwise_mul(identity_fn[Real], identity_fn[Real]))
    is_derivative_fn(square_real,
        pointwise_add(pointwise_mul(identity_fn[Real], constant[Real, Real](Real.1)),
            pointwise_mul(identity_fn[Real], constant[Real, Real](Real.1))))
}

/// The derivative of the square is the pointwise doubled identity.
lemma square_is_derivative_fn {
    is_derivative_fn(square_real, square_dx)
} by {
    square_has_derivative_combinator
    is_derivative_fn(square_real,
        pointwise_add(pointwise_mul(identity_fn[Real], constant[Real, Real](Real.1)),
            pointwise_mul(identity_fn[Real], constant[Real, Real](Real.1))))
    forall(x: Real) {
        square_dx_pointwise_eq(x)
        pointwise_add(pointwise_mul(identity_fn[Real], constant[Real, Real](Real.1)),
            pointwise_mul(identity_fn[Real], constant[Real, Real](Real.1)), x) = square_dx(x)
    }
    function_extensionality(pointwise_add(pointwise_mul(identity_fn[Real], constant[Real, Real](Real.1)),
        pointwise_mul(identity_fn[Real], constant[Real, Real](Real.1))), square_dx)
    pointwise_add(pointwise_mul(identity_fn[Real], constant[Real, Real](Real.1)),
        pointwise_mul(identity_fn[Real], constant[Real, Real](Real.1))) = square_dx
    is_derivative_fn_pointwise_eq(square_real,
        pointwise_add(pointwise_mul(identity_fn[Real], constant[Real, Real](Real.1)),
            pointwise_mul(identity_fn[Real], constant[Real, Real](Real.1))), square_dx)
    is_derivative_fn(square_real, square_dx)
}

/// The cube-derivative combinator agrees with the named derivative function.
lemma cube_dx_pointwise_eq(x: Real) {
    pointwise_add(pointwise_mul(square_real, constant[Real, Real](Real.1)),
        pointwise_mul(identity_fn[Real],
            pointwise_add(pointwise_mul(identity_fn[Real], constant[Real, Real](Real.1)),
                pointwise_mul(identity_fn[Real], constant[Real, Real](Real.1)))), x) = cube_dx(x)
} by {
    pointwise_mul(square_real, constant[Real, Real](Real.1), x) = square_real(x) * Real.1
    square_real(x) = x * x
    pointwise_mul(square_real, constant[Real, Real](Real.1), x) = x * x
    pointwise_mul(identity_fn[Real], pointwise_add(pointwise_mul(identity_fn[Real], constant[Real, Real](Real.1)), pointwise_mul(identity_fn[Real], constant[Real, Real](Real.1))), x) = identity_fn[Real](x) * pointwise_add(pointwise_mul(identity_fn[Real], constant[Real, Real](Real.1)), pointwise_mul(identity_fn[Real], constant[Real, Real](Real.1)), x)
    identity_fn[Real](x) = x
    square_dx_pointwise_eq(x)
    pointwise_add(pointwise_mul(identity_fn[Real], constant[Real, Real](Real.1)),
        pointwise_mul(identity_fn[Real], constant[Real, Real](Real.1)), x) = square_dx(x)
    pointwise_mul(identity_fn[Real], pointwise_add(pointwise_mul(identity_fn[Real], constant[Real, Real](Real.1)), pointwise_mul(identity_fn[Real], constant[Real, Real](Real.1))), x) = x * square_dx(x)
    pointwise_add(pointwise_mul(square_real, constant[Real, Real](Real.1)), pointwise_mul(identity_fn[Real], pointwise_add(pointwise_mul(identity_fn[Real], constant[Real, Real](Real.1)), pointwise_mul(identity_fn[Real], constant[Real, Real](Real.1)))), x) = pointwise_mul(square_real, constant[Real, Real](Real.1), x) + pointwise_mul(identity_fn[Real], pointwise_add(pointwise_mul(identity_fn[Real], constant[Real, Real](Real.1)), pointwise_mul(identity_fn[Real], constant[Real, Real](Real.1))), x)
    pointwise_mul(square_real, constant[Real, Real](Real.1), x) = x * x
    pointwise_mul(identity_fn[Real], pointwise_add(pointwise_mul(identity_fn[Real], constant[Real, Real](Real.1)), pointwise_mul(identity_fn[Real], constant[Real, Real](Real.1))), x) = x * square_dx(x)
    x * x + x * square_dx(x) = x * x + x * square_dx(x)
    pointwise_add(pointwise_mul(square_real, constant[Real, Real](Real.1)), pointwise_mul(identity_fn[Real], pointwise_add(pointwise_mul(identity_fn[Real], constant[Real, Real](Real.1)), pointwise_mul(identity_fn[Real], constant[Real, Real](Real.1)))), x) = x * x + x * square_dx(x)
    square_dx(x) = x * Real.1 + x * Real.1
    x * Real.1 = x
    square_dx(x) = x + x
    x + x = two * x
    square_dx(x) = two * x
    x * square_dx(x) = x * (two * x)
    x * (two * x) = two * (x * x)
    pointwise_add(pointwise_mul(square_real, constant[Real, Real](Real.1)), pointwise_mul(identity_fn[Real], pointwise_add(pointwise_mul(identity_fn[Real], constant[Real, Real](Real.1)), pointwise_mul(identity_fn[Real], constant[Real, Real](Real.1)))), x) = x * x + two * (x * x)
    x * x + two * (x * x) = three * (x * x)
    pointwise_add(pointwise_mul(square_real, constant[Real, Real](Real.1)), pointwise_mul(identity_fn[Real], pointwise_add(pointwise_mul(identity_fn[Real], constant[Real, Real](Real.1)), pointwise_mul(identity_fn[Real], constant[Real, Real](Real.1)))), x) = three * (x * x)
    cube_dx(x) = square_real(x) * Real.1 + x * (x * Real.1 + x * Real.1)
    square_real(x) = x * x
    square_real(x) * Real.1 = x * x
    x * (x * Real.1 + x * Real.1) = x * (x + x)
    x + x = two * x
    x * (x + x) = x * (two * x)
    x * (two * x) = two * (x * x)
    x * (x * Real.1 + x * Real.1) = two * (x * x)
    cube_dx(x) = x * x + two * (x * x)
    x * x + two * (x * x) = three * (x * x)
    cube_dx(x) = three * (x * x)
    pointwise_add(pointwise_mul(square_real, constant[Real, Real](Real.1)),
        pointwise_mul(identity_fn[Real],
            pointwise_add(pointwise_mul(identity_fn[Real], constant[Real, Real](Real.1)),
                pointwise_mul(identity_fn[Real], constant[Real, Real](Real.1)))), x) = cube_dx(x)
}

/// The cube has the iterated product-rule derivative combinator.
lemma cube_has_derivative_combinator {
    is_derivative_fn(cube_real,
        pointwise_add(pointwise_mul(square_real, constant[Real, Real](Real.1)),
            pointwise_mul(identity_fn[Real],
                pointwise_add(pointwise_mul(identity_fn[Real], constant[Real, Real](Real.1)),
                    pointwise_mul(identity_fn[Real], constant[Real, Real](Real.1))))))
} by {
    square_has_derivative_combinator
    is_derivative_fn(square_real,
        pointwise_add(pointwise_mul(identity_fn[Real], constant[Real, Real](Real.1)),
            pointwise_mul(identity_fn[Real], constant[Real, Real](Real.1))))
    derivative_fn_identity
    is_derivative_fn(identity_fn[Real], constant[Real, Real](Real.1))
    derivative_fn_mul(square_real, identity_fn[Real],
        pointwise_add(pointwise_mul(identity_fn[Real], constant[Real, Real](Real.1)),
            pointwise_mul(identity_fn[Real], constant[Real, Real](Real.1))),
        constant[Real, Real](Real.1))
    is_derivative_fn(pointwise_mul(square_real, identity_fn[Real]),
        pointwise_add(pointwise_mul(square_real, constant[Real, Real](Real.1)),
            pointwise_mul(identity_fn[Real],
                pointwise_add(pointwise_mul(identity_fn[Real], constant[Real, Real](Real.1)),
                    pointwise_mul(identity_fn[Real], constant[Real, Real](Real.1))))))
    cube_real_eq_pointwise_mul_square_identity
    cube_real = pointwise_mul(square_real, identity_fn[Real])
    define cube_derivative_pred(h: Real -> Real) -> Bool {
        is_derivative_fn(h,
            pointwise_add(pointwise_mul(square_real, constant[Real, Real](Real.1)),
                pointwise_mul(identity_fn[Real],
                    pointwise_add(pointwise_mul(identity_fn[Real], constant[Real, Real](Real.1)),
                        pointwise_mul(identity_fn[Real], constant[Real, Real](Real.1))))))
    }
    cube_derivative_pred(pointwise_mul(square_real, identity_fn[Real]))
    function_eq_transport_predicate_rev(cube_derivative_pred, cube_real,
        pointwise_mul(square_real, identity_fn[Real]))
    is_derivative_fn(cube_real,
        pointwise_add(pointwise_mul(square_real, constant[Real, Real](Real.1)),
            pointwise_mul(identity_fn[Real],
                pointwise_add(pointwise_mul(identity_fn[Real], constant[Real, Real](Real.1)),
                    pointwise_mul(identity_fn[Real], constant[Real, Real](Real.1))))))
}

/// One third times the cube derivative is the square pointwise.
lemma one_third_times_cube_dx(x: Real) {
    one_third * cube_dx(x) = square_real(x)
} by {
    cube_dx(x) = square_real(x) * Real.1 + x * (x * Real.1 + x * Real.1)
    square_real(x) = x * x
    square_real(x) * Real.1 = x * x
    x * (x * Real.1 + x * Real.1) = x * (x + x)
    x + x = two * x
    x * (x + x) = x * (two * x)
    x * (two * x) = two * (x * x)
    x * (x * Real.1 + x * Real.1) = two * (x * x)
    cube_dx(x) = x * x + two * (x * x)
    x * x + two * (x * x) = three * (x * x)
    cube_dx(x) = three * (x * x)
    one_third * cube_dx(x) = one_third * (three * (x * x))
    one_third * (three * (x * x)) = (one_third * three) * (x * x)
    three_mul_one_third
    three * one_third = Real.1
    one_third * three = Real.1
    (one_third * three) * (x * x) = Real.1 * (x * x)
    Real.1 * (x * x) = x * x
    one_third * (three * (x * x)) = x * x
    one_third * cube_dx(x) = x * x
    square_real(x) = x * x
    one_third * cube_dx(x) = square_real(x)
}

/// The derivative of x -> x^3 / 3 is the square.
lemma cube_third_is_derivative_fn {
    is_derivative_fn(const_mul_left(one_third, cube_real), square_real)
} by {
    cube_has_derivative_combinator
    is_derivative_fn(cube_real,
        pointwise_add(pointwise_mul(square_real, constant[Real, Real](Real.1)),
            pointwise_mul(identity_fn[Real],
                pointwise_add(pointwise_mul(identity_fn[Real], constant[Real, Real](Real.1)),
                    pointwise_mul(identity_fn[Real], constant[Real, Real](Real.1))))))
    derivative_fn_const_mul(one_third, cube_real,
        pointwise_add(pointwise_mul(square_real, constant[Real, Real](Real.1)),
            pointwise_mul(identity_fn[Real],
                pointwise_add(pointwise_mul(identity_fn[Real], constant[Real, Real](Real.1)),
                    pointwise_mul(identity_fn[Real], constant[Real, Real](Real.1))))))
    is_derivative_fn(pointwise_mul(constant[Real, Real](one_third), cube_real),
        pointwise_mul(constant[Real, Real](one_third),
            pointwise_add(pointwise_mul(square_real, constant[Real, Real](Real.1)),
                pointwise_mul(identity_fn[Real],
                    pointwise_add(pointwise_mul(identity_fn[Real], constant[Real, Real](Real.1)),
                        pointwise_mul(identity_fn[Real], constant[Real, Real](Real.1)))))))
    pointwise_mul_constant_left_eq(one_third, cube_real)
    pointwise_mul(constant[Real, Real](one_third), cube_real) = const_mul_left(one_third, cube_real)
    define cube_third_pred(h: Real -> Real) -> Bool {
        is_derivative_fn(h,
            pointwise_mul(constant[Real, Real](one_third),
                pointwise_add(pointwise_mul(square_real, constant[Real, Real](Real.1)),
                    pointwise_mul(identity_fn[Real],
                        pointwise_add(pointwise_mul(identity_fn[Real], constant[Real, Real](Real.1)),
                            pointwise_mul(identity_fn[Real], constant[Real, Real](Real.1)))))))
    }
    cube_third_pred(pointwise_mul(constant[Real, Real](one_third), cube_real))
    function_eq_transport_predicate_rev(cube_third_pred, const_mul_left(one_third, cube_real),
        pointwise_mul(constant[Real, Real](one_third), cube_real))
    is_derivative_fn(const_mul_left(one_third, cube_real),
        pointwise_mul(constant[Real, Real](one_third),
            pointwise_add(pointwise_mul(square_real, constant[Real, Real](Real.1)),
                pointwise_mul(identity_fn[Real],
                    pointwise_add(pointwise_mul(identity_fn[Real], constant[Real, Real](Real.1)),
                        pointwise_mul(identity_fn[Real], constant[Real, Real](Real.1)))))))
    forall(x: Real) {
        cube_dx_pointwise_eq(x)
        pointwise_mul(constant[Real, Real](one_third), pointwise_add(pointwise_mul(square_real, constant[Real, Real](Real.1)), pointwise_mul(identity_fn[Real], pointwise_add(pointwise_mul(identity_fn[Real], constant[Real, Real](Real.1)), pointwise_mul(identity_fn[Real], constant[Real, Real](Real.1))))), x) = one_third * cube_dx(x)
        one_third_times_cube_dx(x)
        one_third * cube_dx(x) = square_real(x)
        pointwise_mul(constant[Real, Real](one_third), pointwise_add(pointwise_mul(square_real, constant[Real, Real](Real.1)), pointwise_mul(identity_fn[Real], pointwise_add(pointwise_mul(identity_fn[Real], constant[Real, Real](Real.1)), pointwise_mul(identity_fn[Real], constant[Real, Real](Real.1))))), x) = square_real(x)
    }
    function_extensionality(pointwise_mul(constant[Real, Real](one_third),
        pointwise_add(pointwise_mul(square_real, constant[Real, Real](Real.1)),
            pointwise_mul(identity_fn[Real],
                pointwise_add(pointwise_mul(identity_fn[Real], constant[Real, Real](Real.1)),
                    pointwise_mul(identity_fn[Real], constant[Real, Real](Real.1)))))),
        square_real)
    pointwise_mul(constant[Real, Real](one_third),
        pointwise_add(pointwise_mul(square_real, constant[Real, Real](Real.1)),
            pointwise_mul(identity_fn[Real],
                pointwise_add(pointwise_mul(identity_fn[Real], constant[Real, Real](Real.1)),
                    pointwise_mul(identity_fn[Real], constant[Real, Real](Real.1)))))) = square_real
    is_derivative_fn_pointwise_eq(const_mul_left(one_third, cube_real),
        pointwise_mul(constant[Real, Real](one_third),
            pointwise_add(pointwise_mul(square_real, constant[Real, Real](Real.1)),
                pointwise_mul(identity_fn[Real],
                    pointwise_add(pointwise_mul(identity_fn[Real], constant[Real, Real](Real.1)),
                        pointwise_mul(identity_fn[Real], constant[Real, Real](Real.1)))))),
        square_real)
    is_derivative_fn(const_mul_left(one_third, cube_real), square_real)
}

/// The function x -> x^3 / 3 is continuous.
lemma cube_third_continuous {
    continuous(const_mul_left(one_third, cube_real))
} by {
    continuous_cube_real
    continuous(cube_real)
    continuous_const_mul_left(one_third, cube_real)
    continuous(const_mul_left(one_third, cube_real))
}

// ---------------------------------------------------------------------------
// The integral of the centered quadratic (x - 1/2)^2 over [0, 1]
// ---------------------------------------------------------------------------

/// The cube-derivative combinator applied after the shift is three times the
/// centered square.
lemma shifted_cube_dx_pointwise(x: Real) {
    compose(pointwise_add(pointwise_mul(square_real, constant[Real, Real](Real.1)),
        pointwise_mul(identity_fn[Real],
            pointwise_add(pointwise_mul(identity_fn[Real], constant[Real, Real](Real.1)),
                pointwise_mul(identity_fn[Real], constant[Real, Real](Real.1))))), unit_shift, x) = three * (unit_shift(x) * unit_shift(x))
} by {
    cube_dx_pointwise_eq(unit_shift(x))
    pointwise_add(pointwise_mul(square_real, constant[Real, Real](Real.1)),
        pointwise_mul(identity_fn[Real],
            pointwise_add(pointwise_mul(identity_fn[Real], constant[Real, Real](Real.1)),
                pointwise_mul(identity_fn[Real], constant[Real, Real](Real.1)))), unit_shift(x)) = cube_dx(unit_shift(x))
    compose(pointwise_add(pointwise_mul(square_real, constant[Real, Real](Real.1)),
        pointwise_mul(identity_fn[Real],
            pointwise_add(pointwise_mul(identity_fn[Real], constant[Real, Real](Real.1)),
                pointwise_mul(identity_fn[Real], constant[Real, Real](Real.1))))), unit_shift, x) = pointwise_add(pointwise_mul(square_real, constant[Real, Real](Real.1)),
            pointwise_mul(identity_fn[Real],
                pointwise_add(pointwise_mul(identity_fn[Real], constant[Real, Real](Real.1)),
                    pointwise_mul(identity_fn[Real], constant[Real, Real](Real.1)))), unit_shift(x))
    compose(pointwise_add(pointwise_mul(square_real, constant[Real, Real](Real.1)),
        pointwise_mul(identity_fn[Real],
            pointwise_add(pointwise_mul(identity_fn[Real], constant[Real, Real](Real.1)),
                pointwise_mul(identity_fn[Real], constant[Real, Real](Real.1))))), unit_shift, x) = cube_dx(unit_shift(x))
    cube_dx(unit_shift(x)) = square_real(unit_shift(x)) * Real.1 + unit_shift(x) * (unit_shift(x) * Real.1 + unit_shift(x) * Real.1)
    square_real(unit_shift(x)) = unit_shift(x) * unit_shift(x)
    square_real(unit_shift(x)) * Real.1 = unit_shift(x) * unit_shift(x)
    unit_shift(x) * (unit_shift(x) * Real.1 + unit_shift(x) * Real.1) = unit_shift(x) * (unit_shift(x) + unit_shift(x))
    unit_shift(x) + unit_shift(x) = two * unit_shift(x)
    unit_shift(x) * (unit_shift(x) + unit_shift(x)) = unit_shift(x) * (two * unit_shift(x))
    unit_shift(x) * (two * unit_shift(x)) = two * (unit_shift(x) * unit_shift(x))
    unit_shift(x) * (unit_shift(x) * Real.1 + unit_shift(x) * Real.1) = two * (unit_shift(x) * unit_shift(x))
    cube_dx(unit_shift(x)) = unit_shift(x) * unit_shift(x) + two * (unit_shift(x) * unit_shift(x))
    unit_shift(x) * unit_shift(x) + two * (unit_shift(x) * unit_shift(x)) = three * (unit_shift(x) * unit_shift(x))
    cube_dx(unit_shift(x)) = three * (unit_shift(x) * unit_shift(x))
    compose(pointwise_add(pointwise_mul(square_real, constant[Real, Real](Real.1)),
        pointwise_mul(identity_fn[Real],
            pointwise_add(pointwise_mul(identity_fn[Real], constant[Real, Real](Real.1)),
                pointwise_mul(identity_fn[Real], constant[Real, Real](Real.1))))), unit_shift, x) = three * (unit_shift(x) * unit_shift(x))
}

/// One third of the shifted cube derivative is the centered square pointwise.
lemma one_third_shifted_cube_dx(x: Real) {
    one_third * compose(pointwise_add(pointwise_mul(square_real, constant[Real, Real](Real.1)),
        pointwise_mul(identity_fn[Real],
            pointwise_add(pointwise_mul(identity_fn[Real], constant[Real, Real](Real.1)),
                pointwise_mul(identity_fn[Real], constant[Real, Real](Real.1))))), unit_shift, x) = centered_square_unit(x)
} by {
    shifted_cube_dx_pointwise(x)
    compose(pointwise_add(pointwise_mul(square_real, constant[Real, Real](Real.1)),
        pointwise_mul(identity_fn[Real],
            pointwise_add(pointwise_mul(identity_fn[Real], constant[Real, Real](Real.1)),
                pointwise_mul(identity_fn[Real], constant[Real, Real](Real.1))))), unit_shift, x) = three * (unit_shift(x) * unit_shift(x))
    one_third * compose(pointwise_add(pointwise_mul(square_real, constant[Real, Real](Real.1)),
        pointwise_mul(identity_fn[Real],
            pointwise_add(pointwise_mul(identity_fn[Real], constant[Real, Real](Real.1)),
                pointwise_mul(identity_fn[Real], constant[Real, Real](Real.1))))), unit_shift, x) = one_third * (three * (unit_shift(x) * unit_shift(x)))
    one_third * (three * (unit_shift(x) * unit_shift(x))) = (one_third * three) * (unit_shift(x) * unit_shift(x))
    three_mul_one_third
    three * one_third = Real.1
    one_third * three = Real.1
    (one_third * three) * (unit_shift(x) * unit_shift(x)) = Real.1 * (unit_shift(x) * unit_shift(x))
    Real.1 * (unit_shift(x) * unit_shift(x)) = unit_shift(x) * unit_shift(x)
    one_third * (three * (unit_shift(x) * unit_shift(x))) = unit_shift(x) * unit_shift(x)
    one_third * compose(pointwise_add(pointwise_mul(square_real, constant[Real, Real](Real.1)),
        pointwise_mul(identity_fn[Real],
            pointwise_add(pointwise_mul(identity_fn[Real], constant[Real, Real](Real.1)),
                pointwise_mul(identity_fn[Real], constant[Real, Real](Real.1))))), unit_shift, x) = unit_shift(x) * unit_shift(x)
    unit_shift(x) = x - Real.one_half
    unit_shift(x) * unit_shift(x) = (x - Real.one_half) * (x - Real.one_half)
    compose(square_real, unit_shift, x) = square_real(unit_shift(x))
    square_real(unit_shift(x)) = unit_shift(x) * unit_shift(x)
    centered_square_unit(x) = compose(square_real, unit_shift, x)
    centered_square_unit(x) = square_real(unit_shift(x))
    centered_square_unit(x) = unit_shift(x) * unit_shift(x)
    centered_square_unit(x) = (x - Real.one_half) * (x - Real.one_half)
    unit_shift(x) * unit_shift(x) = centered_square_unit(x)
    one_third * compose(pointwise_add(pointwise_mul(square_real, constant[Real, Real](Real.1)),
        pointwise_mul(identity_fn[Real],
            pointwise_add(pointwise_mul(identity_fn[Real], constant[Real, Real](Real.1)),
                pointwise_mul(identity_fn[Real], constant[Real, Real](Real.1))))), unit_shift, x) = centered_square_unit(x)
}

/// The shift agrees with the pointwise affine normal form x -> 1 * x + (-1/2).
lemma unit_shift_eq_pointwise_affine {
    unit_shift = pointwise_add(pointwise_mul(constant[Real, Real](Real.1), identity_fn[Real]),
        constant[Real, Real](-Real.one_half))
} by {
    forall(x: Real) {
        pointwise_mul(constant[Real, Real](Real.1), identity_fn[Real], x) = Real.1 * identity_fn[Real](x)
        identity_fn[Real](x) = x
        pointwise_mul(constant[Real, Real](Real.1), identity_fn[Real], x) = Real.1 * x
        Real.1 * x = x
        pointwise_add(pointwise_mul(constant[Real, Real](Real.1), identity_fn[Real]),
            constant[Real, Real](-Real.one_half), x) = x + -Real.one_half
        unit_shift(x) = x - Real.one_half
        x - Real.one_half = x + -Real.one_half
        pointwise_add(pointwise_mul(constant[Real, Real](Real.1), identity_fn[Real]),
            constant[Real, Real](-Real.one_half), x) = unit_shift(x)
    }
    function_extensionality(pointwise_add(pointwise_mul(constant[Real, Real](Real.1), identity_fn[Real]),
        constant[Real, Real](-Real.one_half)), unit_shift)
    pointwise_add(pointwise_mul(constant[Real, Real](Real.1), identity_fn[Real]),
        constant[Real, Real](-Real.one_half)) = unit_shift
}

/// The derivative of one third of the shifted cube is the centered square.
lemma centered_cube_unit_is_derivative_fn {
    is_derivative_fn(const_mul_left(one_third, compose(cube_real, unit_shift)), centered_square_unit)
} by {
    derivative_fn_affine_identity(Real.1, -Real.one_half)
    is_derivative_fn(pointwise_add(pointwise_mul(constant[Real, Real](Real.1), identity_fn[Real]),
        constant[Real, Real](-Real.one_half)), constant[Real, Real](Real.1))
    unit_shift_eq_pointwise_affine
    pointwise_add(pointwise_mul(constant[Real, Real](Real.1), identity_fn[Real]),
        constant[Real, Real](-Real.one_half)) = unit_shift
    define unit_shift_pred(h: Real -> Real) -> Bool {
        is_derivative_fn(h, constant[Real, Real](Real.1))
    }
    unit_shift_pred(pointwise_add(pointwise_mul(constant[Real, Real](Real.1), identity_fn[Real]),
        constant[Real, Real](-Real.one_half)))
    function_eq_transport_predicate_rev(unit_shift_pred, unit_shift,
        pointwise_add(pointwise_mul(constant[Real, Real](Real.1), identity_fn[Real]),
            constant[Real, Real](-Real.one_half)))
    is_derivative_fn(unit_shift, constant[Real, Real](Real.1))
    square_has_derivative_combinator
    is_derivative_fn(square_real,
        pointwise_add(pointwise_mul(identity_fn[Real], constant[Real, Real](Real.1)),
            pointwise_mul(identity_fn[Real], constant[Real, Real](Real.1))))
    derivative_fn_compose(unit_shift, square_real, constant[Real, Real](Real.1),
        pointwise_add(pointwise_mul(identity_fn[Real], constant[Real, Real](Real.1)),
            pointwise_mul(identity_fn[Real], constant[Real, Real](Real.1))))
    is_derivative_fn(compose(square_real, unit_shift),
        pointwise_mul(compose(pointwise_add(pointwise_mul(identity_fn[Real], constant[Real, Real](Real.1)),
            pointwise_mul(identity_fn[Real], constant[Real, Real](Real.1))), unit_shift),
            constant[Real, Real](Real.1)))
    cube_has_derivative_combinator
    is_derivative_fn(cube_real,
        pointwise_add(pointwise_mul(square_real, constant[Real, Real](Real.1)),
            pointwise_mul(identity_fn[Real],
                pointwise_add(pointwise_mul(identity_fn[Real], constant[Real, Real](Real.1)),
                    pointwise_mul(identity_fn[Real], constant[Real, Real](Real.1))))))
    derivative_fn_compose(unit_shift, cube_real, constant[Real, Real](Real.1),
        pointwise_add(pointwise_mul(square_real, constant[Real, Real](Real.1)),
            pointwise_mul(identity_fn[Real],
                pointwise_add(pointwise_mul(identity_fn[Real], constant[Real, Real](Real.1)),
                    pointwise_mul(identity_fn[Real], constant[Real, Real](Real.1))))))
    is_derivative_fn(compose(cube_real, unit_shift),
        pointwise_mul(compose(pointwise_add(pointwise_mul(square_real, constant[Real, Real](Real.1)),
            pointwise_mul(identity_fn[Real],
                pointwise_add(pointwise_mul(identity_fn[Real], constant[Real, Real](Real.1)),
                    pointwise_mul(identity_fn[Real], constant[Real, Real](Real.1))))), unit_shift),
            constant[Real, Real](Real.1)))
    derivative_fn_const_mul(one_third, compose(cube_real, unit_shift),
        pointwise_mul(compose(pointwise_add(pointwise_mul(square_real, constant[Real, Real](Real.1)),
            pointwise_mul(identity_fn[Real],
                pointwise_add(pointwise_mul(identity_fn[Real], constant[Real, Real](Real.1)),
                    pointwise_mul(identity_fn[Real], constant[Real, Real](Real.1))))), unit_shift),
            constant[Real, Real](Real.1)))
    is_derivative_fn(pointwise_mul(constant[Real, Real](one_third), compose(cube_real, unit_shift)),
        pointwise_mul(constant[Real, Real](one_third),
            pointwise_mul(compose(pointwise_add(pointwise_mul(square_real, constant[Real, Real](Real.1)),
                pointwise_mul(identity_fn[Real],
                    pointwise_add(pointwise_mul(identity_fn[Real], constant[Real, Real](Real.1)),
                        pointwise_mul(identity_fn[Real], constant[Real, Real](Real.1))))), unit_shift),
                constant[Real, Real](Real.1))))
    pointwise_mul_constant_left_eq(one_third, compose(cube_real, unit_shift))
    pointwise_mul(constant[Real, Real](one_third), compose(cube_real, unit_shift)) = const_mul_left(one_third, compose(cube_real, unit_shift))
    define centered_cube_pred(h: Real -> Real) -> Bool {
        is_derivative_fn(h,
            pointwise_mul(constant[Real, Real](one_third),
                pointwise_mul(compose(pointwise_add(pointwise_mul(square_real, constant[Real, Real](Real.1)),
                    pointwise_mul(identity_fn[Real],
                        pointwise_add(pointwise_mul(identity_fn[Real], constant[Real, Real](Real.1)),
                            pointwise_mul(identity_fn[Real], constant[Real, Real](Real.1))))), unit_shift),
                    constant[Real, Real](Real.1))))
    }
    centered_cube_pred(pointwise_mul(constant[Real, Real](one_third), compose(cube_real, unit_shift)))
    function_eq_transport_predicate_rev(centered_cube_pred, const_mul_left(one_third, compose(cube_real, unit_shift)),
        pointwise_mul(constant[Real, Real](one_third), compose(cube_real, unit_shift)))
    is_derivative_fn(const_mul_left(one_third, compose(cube_real, unit_shift)),
        pointwise_mul(constant[Real, Real](one_third),
            pointwise_mul(compose(pointwise_add(pointwise_mul(square_real, constant[Real, Real](Real.1)),
                pointwise_mul(identity_fn[Real],
                    pointwise_add(pointwise_mul(identity_fn[Real], constant[Real, Real](Real.1)),
                        pointwise_mul(identity_fn[Real], constant[Real, Real](Real.1))))), unit_shift),
                constant[Real, Real](Real.1))))
    forall(x: Real) {
        constant[Real, Real](one_third, x) = one_third
        pointwise_mul(constant[Real, Real](one_third),
            pointwise_mul(compose(pointwise_add(pointwise_mul(square_real, constant[Real, Real](Real.1)),
                pointwise_mul(identity_fn[Real],
                    pointwise_add(pointwise_mul(identity_fn[Real], constant[Real, Real](Real.1)),
                        pointwise_mul(identity_fn[Real], constant[Real, Real](Real.1))))), unit_shift),
                constant[Real, Real](Real.1)), x) = one_third * pointwise_mul(compose(pointwise_add(pointwise_mul(square_real, constant[Real, Real](Real.1)),
                pointwise_mul(identity_fn[Real],
                    pointwise_add(pointwise_mul(identity_fn[Real], constant[Real, Real](Real.1)),
                        pointwise_mul(identity_fn[Real], constant[Real, Real](Real.1))))), unit_shift),
                constant[Real, Real](Real.1), x)
        pointwise_mul(compose(pointwise_add(pointwise_mul(square_real, constant[Real, Real](Real.1)),
            pointwise_mul(identity_fn[Real],
                pointwise_add(pointwise_mul(identity_fn[Real], constant[Real, Real](Real.1)),
                    pointwise_mul(identity_fn[Real], constant[Real, Real](Real.1))))), unit_shift),
            constant[Real, Real](Real.1), x) = compose(pointwise_add(pointwise_mul(square_real, constant[Real, Real](Real.1)),
                pointwise_mul(identity_fn[Real],
                    pointwise_add(pointwise_mul(identity_fn[Real], constant[Real, Real](Real.1)),
                        pointwise_mul(identity_fn[Real], constant[Real, Real](Real.1))))), unit_shift, x) * Real.1
        compose(pointwise_add(pointwise_mul(square_real, constant[Real, Real](Real.1)),
            pointwise_mul(identity_fn[Real],
                pointwise_add(pointwise_mul(identity_fn[Real], constant[Real, Real](Real.1)),
                    pointwise_mul(identity_fn[Real], constant[Real, Real](Real.1))))), unit_shift, x) * Real.1 = compose(pointwise_add(pointwise_mul(square_real, constant[Real, Real](Real.1)),
                pointwise_mul(identity_fn[Real],
                    pointwise_add(pointwise_mul(identity_fn[Real], constant[Real, Real](Real.1)),
                        pointwise_mul(identity_fn[Real], constant[Real, Real](Real.1))))), unit_shift, x)
        pointwise_mul(constant[Real, Real](one_third),
            pointwise_mul(compose(pointwise_add(pointwise_mul(square_real, constant[Real, Real](Real.1)),
                pointwise_mul(identity_fn[Real],
                    pointwise_add(pointwise_mul(identity_fn[Real], constant[Real, Real](Real.1)),
                        pointwise_mul(identity_fn[Real], constant[Real, Real](Real.1))))), unit_shift),
                constant[Real, Real](Real.1)), x) = one_third * (compose(pointwise_add(pointwise_mul(square_real, constant[Real, Real](Real.1)),
                pointwise_mul(identity_fn[Real],
                    pointwise_add(pointwise_mul(identity_fn[Real], constant[Real, Real](Real.1)),
                        pointwise_mul(identity_fn[Real], constant[Real, Real](Real.1))))), unit_shift, x) * Real.1)
        compose(pointwise_add(pointwise_mul(square_real, constant[Real, Real](Real.1)),
            pointwise_mul(identity_fn[Real],
                pointwise_add(pointwise_mul(identity_fn[Real], constant[Real, Real](Real.1)),
                    pointwise_mul(identity_fn[Real], constant[Real, Real](Real.1))))), unit_shift, x) * Real.1 = compose(pointwise_add(pointwise_mul(square_real, constant[Real, Real](Real.1)),
                pointwise_mul(identity_fn[Real],
                    pointwise_add(pointwise_mul(identity_fn[Real], constant[Real, Real](Real.1)),
                        pointwise_mul(identity_fn[Real], constant[Real, Real](Real.1))))), unit_shift, x)
        pointwise_mul(constant[Real, Real](one_third),
            pointwise_mul(compose(pointwise_add(pointwise_mul(square_real, constant[Real, Real](Real.1)),
                pointwise_mul(identity_fn[Real],
                    pointwise_add(pointwise_mul(identity_fn[Real], constant[Real, Real](Real.1)),
                        pointwise_mul(identity_fn[Real], constant[Real, Real](Real.1))))), unit_shift),
                constant[Real, Real](Real.1)), x) = one_third * compose(pointwise_add(pointwise_mul(square_real, constant[Real, Real](Real.1)),
                pointwise_mul(identity_fn[Real],
                    pointwise_add(pointwise_mul(identity_fn[Real], constant[Real, Real](Real.1)),
                        pointwise_mul(identity_fn[Real], constant[Real, Real](Real.1))))), unit_shift, x)
        one_third_shifted_cube_dx(x)
        one_third * compose(pointwise_add(pointwise_mul(square_real, constant[Real, Real](Real.1)),
            pointwise_mul(identity_fn[Real],
                pointwise_add(pointwise_mul(identity_fn[Real], constant[Real, Real](Real.1)),
                    pointwise_mul(identity_fn[Real], constant[Real, Real](Real.1))))), unit_shift, x) = centered_square_unit(x)
        pointwise_mul(constant[Real, Real](one_third),
            pointwise_mul(compose(pointwise_add(pointwise_mul(square_real, constant[Real, Real](Real.1)),
                pointwise_mul(identity_fn[Real],
                    pointwise_add(pointwise_mul(identity_fn[Real], constant[Real, Real](Real.1)),
                        pointwise_mul(identity_fn[Real], constant[Real, Real](Real.1))))), unit_shift),
                constant[Real, Real](Real.1)), x) = centered_square_unit(x)
    }
    function_extensionality(pointwise_mul(constant[Real, Real](one_third),
        pointwise_mul(compose(pointwise_add(pointwise_mul(square_real, constant[Real, Real](Real.1)),
            pointwise_mul(identity_fn[Real],
                pointwise_add(pointwise_mul(identity_fn[Real], constant[Real, Real](Real.1)),
                    pointwise_mul(identity_fn[Real], constant[Real, Real](Real.1))))), unit_shift),
            constant[Real, Real](Real.1))),
        centered_square_unit)
    pointwise_mul(constant[Real, Real](one_third),
        pointwise_mul(compose(pointwise_add(pointwise_mul(square_real, constant[Real, Real](Real.1)),
            pointwise_mul(identity_fn[Real],
                pointwise_add(pointwise_mul(identity_fn[Real], constant[Real, Real](Real.1)),
                    pointwise_mul(identity_fn[Real], constant[Real, Real](Real.1))))), unit_shift),
            constant[Real, Real](Real.1))) = centered_square_unit
    is_derivative_fn_pointwise_eq(const_mul_left(one_third, compose(cube_real, unit_shift)),
        pointwise_mul(constant[Real, Real](one_third),
            pointwise_mul(compose(pointwise_add(pointwise_mul(square_real, constant[Real, Real](Real.1)),
                pointwise_mul(identity_fn[Real],
                    pointwise_add(pointwise_mul(identity_fn[Real], constant[Real, Real](Real.1)),
                        pointwise_mul(identity_fn[Real], constant[Real, Real](Real.1))))), unit_shift),
                constant[Real, Real](Real.1))),
        centered_square_unit)
    is_derivative_fn(const_mul_left(one_third, compose(cube_real, unit_shift)), centered_square_unit)
}

/// The shift x -> x - 1/2 is continuous.
lemma unit_shift_continuous {
    continuous(unit_shift)
} by {
    identity_function_is_continuous
    continuous(identity_fn[Real])
    constant_function_is_continuous(Real.1)
    continuous(constant[Real, Real](Real.1))
    continuous_pointwise_mul(constant[Real, Real](Real.1), identity_fn[Real])
    continuous(pointwise_mul(constant[Real, Real](Real.1), identity_fn[Real]))
    constant_function_is_continuous(-Real.one_half)
    continuous(constant[Real, Real](-Real.one_half))
    continuous_pointwise_add(pointwise_mul(constant[Real, Real](Real.1), identity_fn[Real]),
        constant[Real, Real](-Real.one_half))
    continuous(pointwise_add(pointwise_mul(constant[Real, Real](Real.1), identity_fn[Real]),
        constant[Real, Real](-Real.one_half)))
    unit_shift_eq_pointwise_affine
    define unit_shift_cont_pred(h: Real -> Real) -> Bool {
        continuous(h)
    }
    unit_shift_cont_pred(pointwise_add(pointwise_mul(constant[Real, Real](Real.1), identity_fn[Real]),
        constant[Real, Real](-Real.one_half)))
    function_eq_transport_predicate_rev(unit_shift_cont_pred, unit_shift,
        pointwise_add(pointwise_mul(constant[Real, Real](Real.1), identity_fn[Real]),
            constant[Real, Real](-Real.one_half)))
    continuous(unit_shift)
}

/// The antiderivative of the centered square is continuous.
lemma centered_cube_continuous {
    continuous(const_mul_left(one_third, compose(cube_real, unit_shift)))
} by {
    continuous_cube_real
    continuous(cube_real)
    unit_shift_continuous
    continuous(unit_shift)
    continuous_compose(cube_real, unit_shift)
    continuous(compose(cube_real, unit_shift))
    continuous_const_mul_left(one_third, compose(cube_real, unit_shift))
    continuous(const_mul_left(one_third, compose(cube_real, unit_shift)))
}


/// Difference of squares factorization.
lemma sq_sub_sq_factor(a: Real, b: Real) {
    a * a - b * b = (a - b) * (a + b)
} by {
    (a - b) * (a + b) = (a - b) * a + (a - b) * b
    (a - b) * a = a * a - b * a
    (a - b) * b = a * b - b * b
    (a - b) * (a + b) = a * a - b * a + (a * b - b * b)
    real_mul_comm(a, b)
    a * b = b * a
    a * a - b * a + (a * b - b * b) = a * a - b * a + a * b - b * b
    a * b = b * a
    -b * a + a * b = -b * a + b * a
    add_comm(-b * a, b * a)
    -b * a + b * a = b * a + -(b * a)
    b * a + -(b * a) = Real.0
    -b * a + b * a = Real.0
    a * a - b * a + a * b - b * b = a * a - b * b
    (a - b) * (a + b) = a * a - b * b
    a * a - b * b = (a - b) * (a + b)
}

/// Square of a difference expansion.
lemma sq_sub_expand(t: Real, a: Real) {
    (t - a) * (t - a) = t * t - (a * t + a * t) + a * a
} by {
    (t - a) * (t - a) = (t - a) * t - (t - a) * a
    (t - a) * t = t * t - a * t
    (t - a) * a = t * a - a * a
    (t - a) * (t - a) = t * t - a * t - (t * a - a * a)
    neg_distrib(t * a, -(a * a))
    -(t * a + -(a * a)) = -(t * a) + -(-(a * a))
    -(-(a * a)) = a * a
    -(t * a + -(a * a)) = -(t * a) + a * a
    t * a - a * a = t * a + -(a * a)
    -(t * a - a * a) = -(t * a) + a * a
    t * t - a * t - (t * a - a * a) = t * t - a * t + -(t * a - a * a)
    t * t - a * t + -(t * a - a * a) = t * t - a * t + (-(t * a) + a * a)
    t * t - a * t - (t * a - a * a) = t * t - a * t + (-(t * a) + a * a)
    t * t - a * t + (-(t * a) + a * a) = t * t - a * t - t * a + a * a
    t * t - a * t - t * a + a * a = t * t - (a * t + t * a) + a * a
    real_mul_comm(t, a)
    t * a = a * t
    a * t + t * a = a * t + a * t
    t * t - (a * t + t * a) + a * a = t * t - (a * t + a * t) + a * a
    (t - a) * (t - a) = t * t - (a * t + a * t) + a * a
}

/// Doubling a difference distributes.
lemma two_mul_sub_distrib(c: Real, d: Real) {
    two * (c - d) = two * c - two * d
} by {
    real_mul_comm(two, c - d)
    two * (c - d) = (c - d) * two
    mul_sub_distrib_left(c, d, two)
    (c - d) * two = c * two - d * two
    real_mul_comm(c, two)
    c * two = two * c
    real_mul_comm(d, two)
    d * two = two * d
    (c - d) * two = two * c - two * d
    two * (c - d) = two * c - two * d
}

/// On [0, 1] the centered square is nonnegative.
lemma centered_square_lower_bound_unit(t: Real) {
    interval_contains(Real.0, Real.1, t) implies Real.0 <= centered_square_unit(t)
} by {
    if interval_contains(Real.0, Real.1, t) {
        centered_square_unit(t) = (t - Real.one_half) * (t - Real.one_half)
        square_nonneg(t - Real.one_half)
        Real.0 <= (t - Real.one_half) * (t - Real.one_half)
        Real.0 <= centered_square_unit(t)
    }
}

/// On [0, 1] the centered square is bounded above by one quarter.
lemma centered_square_upper_bound_unit(t: Real) {
    interval_contains(Real.0, Real.1, t) implies centered_square_unit(t) <= Real.one_half * Real.one_half
} by {
    if interval_contains(Real.0, Real.1, t) {
        centered_square_unit(t) = (t - Real.one_half) * (t - Real.one_half)
        interval_contains_right(Real.0, Real.1, t)
        t <= Real.1
        interval_contains_left(Real.0, Real.1, t)
        Real.0 <= t
        mul_le_mul_of_nonneg_right(t, Real.1, t)
        t * t <= t * Real.1
        t * Real.1 = t
        t * t <= t
        sq_sub_expand(t, Real.one_half)
        (t - Real.one_half) * (t - Real.one_half) = t * t - (Real.one_half * t + Real.one_half * t) + Real.one_half * Real.one_half
        Real.one_half * t + Real.one_half * t = (Real.one_half + Real.one_half) * t
        one_half_plus_one_half
        Real.one_half + Real.one_half = Real.1
        (Real.one_half + Real.one_half) * t = Real.1 * t
        Real.1 * t = t
        Real.one_half * t + Real.one_half * t = t
        (t - Real.one_half) * (t - Real.one_half) = t * t - t + Real.one_half * Real.one_half
        t * t - t <= Real.0
        add_le_add_right(t * t - t, Real.0, Real.one_half * Real.one_half)
        t * t - t + Real.one_half * Real.one_half <= Real.0 + Real.one_half * Real.one_half
        Real.0 + Real.one_half * Real.one_half = Real.one_half * Real.one_half
        t * t - t + Real.one_half * Real.one_half <= Real.one_half * Real.one_half
        (t - Real.one_half) * (t - Real.one_half) <= Real.one_half * Real.one_half
        centered_square_unit(t) <= Real.one_half * Real.one_half
    }
}

/// A real bounded above and whose negation is bounded above by c has absolute
/// value at most c.
lemma abs_le_of_lte_and_neg_lte(w: Real, c: Real) {
    w <= c and -w <= c implies w.abs <= c
} by {
    if w.is_negative {
        w.abs = -w
        -w <= c
        w.abs <= c
    }
    if not w.is_negative {
        w.abs = w
        w <= c
        w.abs <= c
    }
    w.is_negative or not w.is_negative
    w.abs <= c
}

/// The centered square is one-Lipschitz on [0, 1].
lemma centered_square_lipschitz_unit {
    forall(u: Real, v: Real) {
        interval_contains(Real.0, Real.1, u) and interval_contains(Real.0, Real.1, v)
        implies (centered_square_unit(u) - centered_square_unit(v)).abs <= Real.1 * (u - v).abs
    }
} by {
    forall(u: Real, v: Real) {
        if interval_contains(Real.0, Real.1, u) and interval_contains(Real.0, Real.1, v) {
            centered_square_unit(u) = (u - Real.one_half) * (u - Real.one_half)
            centered_square_unit(v) = (v - Real.one_half) * (v - Real.one_half)
            sq_sub_sq_factor(u - Real.one_half, v - Real.one_half)
            (u - Real.one_half) * (u - Real.one_half) - (v - Real.one_half) * (v - Real.one_half) = (u - Real.one_half - (v - Real.one_half)) * (u - Real.one_half + (v - Real.one_half))
            u - Real.one_half - (v - Real.one_half) = u - v
            u - Real.one_half + (v - Real.one_half) = u + v - Real.1
            (u - Real.one_half - (v - Real.one_half)) * (u - Real.one_half + (v - Real.one_half)) = (u - v) * (u + v - Real.1)
            (u - Real.one_half) * (u - Real.one_half) - (v - Real.one_half) * (v - Real.one_half) = (u - v) * (u + v - Real.1)
            centered_square_unit(u) - centered_square_unit(v) = (u - v) * (u + v - Real.1)
            mul_abs(u - v, u + v - Real.1)
            ((u - v) * (u + v - Real.1)).abs = (u - v).abs * (u + v - Real.1).abs
            (centered_square_unit(u) - centered_square_unit(v)).abs = (u - v).abs * (u + v - Real.1).abs
            interval_contains_left(Real.0, Real.1, u)
            Real.0 <= u
            interval_contains_left(Real.0, Real.1, v)
            Real.0 <= v
            interval_contains_right(Real.0, Real.1, u)
            u <= Real.1
            interval_contains_right(Real.0, Real.1, v)
            v <= Real.1
            add_le_add(Real.0, u, Real.0, v)
            Real.0 + Real.0 <= u + v
            Real.0 + Real.0 = Real.0
            Real.0 <= u + v
            add_le_add(u, Real.1, v, Real.1)
            u + v <= Real.1 + Real.1
            Real.1 + Real.1 = two
            u + v <= two
            add_le_add_right(Real.0, u + v, -Real.1)
            Real.0 + -Real.1 <= u + v + -Real.1
            Real.0 + -Real.1 = -Real.1
            u + v - Real.1 = u + v + -Real.1
            -Real.1 <= u + v - Real.1
            add_le_add_right(u + v, two, -Real.1)
            u + v + -Real.1 <= two + -Real.1
            two = Real.1 + Real.1
            two + -Real.1 = Real.1 + Real.1 + -Real.1
            Real.1 + Real.1 + -Real.1 = Real.1 + (Real.1 + -Real.1)
            Real.1 + -Real.1 = Real.0
            Real.1 + (Real.1 + -Real.1) = Real.1 + Real.0
            Real.1 + Real.0 = Real.1
            two + -Real.1 = Real.1
            u + v - Real.1 = u + v + -Real.1
            u + v + -Real.1 <= Real.1
            u + v - Real.1 <= Real.1
            -Real.1 <= u + v - Real.1 and u + v - Real.1 <= Real.1
            neg_lte_flip(-Real.1, u + v - Real.1)
            -(u + v - Real.1) <= -(-Real.1)
            -(-Real.1) = Real.1
            -(u + v - Real.1) <= Real.1
            u + v - Real.1 <= Real.1 and -(u + v - Real.1) <= Real.1
            abs_le_of_lte_and_neg_lte(u + v - Real.1, Real.1)
            (u + v - Real.1).abs <= Real.1
            abs_gte_zero(u - v)
            Real.0 <= (u - v).abs
            mul_le_mul_of_nonneg_right((u + v - Real.1).abs, Real.1, (u - v).abs)
            (u + v - Real.1).abs * (u - v).abs <= Real.1 * (u - v).abs
            real_mul_comm((u + v - Real.1).abs, (u - v).abs)
            (u - v).abs * (u + v - Real.1).abs <= Real.1 * (u - v).abs
            (centered_square_unit(u) - centered_square_unit(v)).abs <= Real.1 * (u - v).abs
        }
    }
}

// ---------------------------------------------------------------------------
// The integral of the centered quadratic (x - 1/2)^2 over [0, 1]
// ---------------------------------------------------------------------------

/// The centered square is integrable on [0, 1].
lemma centered_square_integrable_unit {
    is_integrable(centered_square_unit, Real.0, Real.1)
} by {
    centered_cube_continuous
    continuous(const_mul_left(one_third, compose(cube_real, unit_shift)))
    centered_cube_unit_is_derivative_fn
    is_derivative_fn(const_mul_left(one_third, compose(cube_real, unit_shift)), centered_square_unit)
    centered_square_lipschitz_unit
    forall(t: Real) {
        centered_square_lower_bound_unit(t)
        interval_contains(Real.0, Real.1, t) implies Real.0 <= centered_square_unit(t)
    }
    forall(t: Real) { interval_contains(Real.0, Real.1, t) implies Real.0 <= centered_square_unit(t) }
    forall(t: Real) {
        centered_square_upper_bound_unit(t)
        interval_contains(Real.0, Real.1, t) implies centered_square_unit(t) <= Real.one_half * Real.one_half
    }
    forall(t: Real) { interval_contains(Real.0, Real.1, t) implies centered_square_unit(t) <= Real.one_half * Real.one_half }
    Real.0 <= Real.1
    zero_is_smaller_than_one[Real]
    Real.0 < Real.1
    lt_imp_lte(Real.0, Real.1)
    eq_true_intro(continuous(const_mul_left(one_third, compose(cube_real, unit_shift))))
    (continuous(const_mul_left(one_third, compose(cube_real, unit_shift)))) = true
    eq_true_intro(is_derivative_fn(const_mul_left(one_third, compose(cube_real, unit_shift)), centered_square_unit))
    (is_derivative_fn(const_mul_left(one_third, compose(cube_real, unit_shift)), centered_square_unit)) = true
    eq_true_intro(forall(u: Real, v: Real) {
        interval_contains(Real.0, Real.1, u) and interval_contains(Real.0, Real.1, v)
        implies (centered_square_unit(u) - centered_square_unit(v)).abs <= Real.1 * (u - v).abs
    })
    (forall(u: Real, v: Real) {
        interval_contains(Real.0, Real.1, u) and interval_contains(Real.0, Real.1, v)
        implies (centered_square_unit(u) - centered_square_unit(v)).abs <= Real.1 * (u - v).abs
    }) = true
    eq_true_intro(forall(t: Real) { interval_contains(Real.0, Real.1, t) implies Real.0 <= centered_square_unit(t) })
    (forall(t: Real) { interval_contains(Real.0, Real.1, t) implies Real.0 <= centered_square_unit(t) }) = true
    eq_true_intro(forall(t: Real) { interval_contains(Real.0, Real.1, t) implies centered_square_unit(t) <= Real.one_half * Real.one_half })
    (forall(t: Real) { interval_contains(Real.0, Real.1, t) implies centered_square_unit(t) <= Real.one_half * Real.one_half }) = true
    Real.0 <= Real.1 and Real.0 <= Real.1 and
        continuous(const_mul_left(one_third, compose(cube_real, unit_shift))) and
        is_derivative_fn(const_mul_left(one_third, compose(cube_real, unit_shift)), centered_square_unit) and
        (forall(u: Real, v: Real) {
            interval_contains(Real.0, Real.1, u) and interval_contains(Real.0, Real.1, v)
            implies (centered_square_unit(u) - centered_square_unit(v)).abs <= Real.1 * (u - v).abs
        }) and
        (forall(t: Real) { interval_contains(Real.0, Real.1, t) implies Real.0 <= centered_square_unit(t) }) and
        (forall(t: Real) { interval_contains(Real.0, Real.1, t) implies centered_square_unit(t) <= Real.one_half * Real.one_half })
    fn_integrable_gen(centered_square_unit, const_mul_left(one_third, compose(cube_real, unit_shift)),
        Real.0, Real.1, Real.1, Real.0, Real.one_half * Real.one_half)
    is_integrable(centered_square_unit, Real.0, Real.1)
}

/// The integral of the centered square over [0, 1] is one twelfth.
theorem integral_centered_square_unit {
    integral(centered_square_unit, Real.0, Real.1) = one_third * (Real.one_half * Real.one_half)
} by {
    centered_square_integrable_unit
    is_integrable(centered_square_unit, Real.0, Real.1)
    centered_cube_continuous
    continuous(const_mul_left(one_third, compose(cube_real, unit_shift)))
    centered_cube_unit_is_derivative_fn
    is_derivative_fn(const_mul_left(one_third, compose(cube_real, unit_shift)), centered_square_unit)
    forall(t: Real) {
        centered_square_lower_bound_unit(t)
        interval_contains(Real.0, Real.1, t) implies Real.0 <= centered_square_unit(t)
    }
    forall(t: Real) { interval_contains(Real.0, Real.1, t) implies Real.0 <= centered_square_unit(t) }
    forall(t: Real) {
        centered_square_upper_bound_unit(t)
        interval_contains(Real.0, Real.1, t) implies centered_square_unit(t) <= Real.one_half * Real.one_half
    }
    forall(t: Real) { interval_contains(Real.0, Real.1, t) implies centered_square_unit(t) <= Real.one_half * Real.one_half }
    Real.0 <= Real.1
    zero_is_smaller_than_one[Real]
    Real.0 < Real.1
    lt_imp_lte(Real.0, Real.1)
    eq_true_intro(continuous(const_mul_left(one_third, compose(cube_real, unit_shift))))
    (continuous(const_mul_left(one_third, compose(cube_real, unit_shift)))) = true
    eq_true_intro(is_derivative_fn(const_mul_left(one_third, compose(cube_real, unit_shift)), centered_square_unit))
    (is_derivative_fn(const_mul_left(one_third, compose(cube_real, unit_shift)), centered_square_unit)) = true
    eq_true_intro(forall(t: Real) { interval_contains(Real.0, Real.1, t) implies Real.0 <= centered_square_unit(t) })
    (forall(t: Real) { interval_contains(Real.0, Real.1, t) implies Real.0 <= centered_square_unit(t) }) = true
    eq_true_intro(forall(t: Real) { interval_contains(Real.0, Real.1, t) implies centered_square_unit(t) <= Real.one_half * Real.one_half })
    (forall(t: Real) { interval_contains(Real.0, Real.1, t) implies centered_square_unit(t) <= Real.one_half * Real.one_half }) = true
    Real.0 <= Real.1 and is_integrable(centered_square_unit, Real.0, Real.1) and
        continuous(const_mul_left(one_third, compose(cube_real, unit_shift))) and
        is_derivative_fn(const_mul_left(one_third, compose(cube_real, unit_shift)), centered_square_unit) and
        (forall(t: Real) { interval_contains(Real.0, Real.1, t) implies Real.0 <= centered_square_unit(t) }) and
        (forall(t: Real) { interval_contains(Real.0, Real.1, t) implies centered_square_unit(t) <= Real.one_half * Real.one_half })
    ftc2_general(centered_square_unit, const_mul_left(one_third, compose(cube_real, unit_shift)),
        Real.0, Real.1, Real.0, Real.one_half * Real.one_half)
    integral(centered_square_unit, Real.0, Real.1) =
        const_mul_left(one_third, compose(cube_real, unit_shift), Real.1) - const_mul_left(one_third, compose(cube_real, unit_shift), Real.0)
    const_mul_left(one_third, compose(cube_real, unit_shift), Real.1) = one_third * cube_real(unit_shift(Real.1))
    const_mul_left(one_third, compose(cube_real, unit_shift), Real.0) = one_third * cube_real(unit_shift(Real.0))
    unit_shift(Real.1) = Real.1 - Real.one_half
    Real.1 - Real.one_half = Real.one_half
    unit_shift(Real.1) = Real.one_half
    unit_shift(Real.0) = Real.0 - Real.one_half
    Real.0 - Real.one_half = -Real.one_half
    unit_shift(Real.0) = -Real.one_half
    cube_real(unit_shift(Real.1)) = cube_real(Real.one_half)
    cube_real(Real.one_half) = Real.one_half * Real.one_half * Real.one_half
    cube_real(unit_shift(Real.0)) = cube_real(-Real.one_half)
    cube_real(-Real.one_half) = (-Real.one_half) * (-Real.one_half) * (-Real.one_half)
    mul_neg_neg(Real.one_half, Real.one_half)
    (-Real.one_half) * (-Real.one_half) = Real.one_half * Real.one_half
    (-Real.one_half) * (-Real.one_half) * (-Real.one_half) = (Real.one_half * Real.one_half) * (-Real.one_half)
    (Real.one_half * Real.one_half) * (-Real.one_half) = -((Real.one_half * Real.one_half) * Real.one_half)
    (Real.one_half * Real.one_half) * Real.one_half = Real.one_half * Real.one_half * Real.one_half
    cube_real(-Real.one_half) = -(Real.one_half * Real.one_half * Real.one_half)
    const_mul_left(one_third, compose(cube_real, unit_shift), Real.1) = one_third * (Real.one_half * Real.one_half * Real.one_half)
    const_mul_left(one_third, compose(cube_real, unit_shift), Real.0) = one_third * (-(Real.one_half * Real.one_half * Real.one_half))
    one_third * (-(Real.one_half * Real.one_half * Real.one_half)) = -(one_third * (Real.one_half * Real.one_half * Real.one_half))
    integral(centered_square_unit, Real.0, Real.1) =
        one_third * (Real.one_half * Real.one_half * Real.one_half) - (-(one_third * (Real.one_half * Real.one_half * Real.one_half)))
    one_third * (Real.one_half * Real.one_half * Real.one_half) - (-(one_third * (Real.one_half * Real.one_half * Real.one_half))) =
        one_third * (Real.one_half * Real.one_half * Real.one_half) + one_third * (Real.one_half * Real.one_half * Real.one_half)
    one_third * (Real.one_half * Real.one_half * Real.one_half) + one_third * (Real.one_half * Real.one_half * Real.one_half) =
        one_third * ((Real.one_half * Real.one_half * Real.one_half) + (Real.one_half * Real.one_half * Real.one_half))
    (Real.one_half * Real.one_half * Real.one_half) + (Real.one_half * Real.one_half * Real.one_half) =
        Real.one_half * Real.one_half
    one_third * ((Real.one_half * Real.one_half * Real.one_half) + (Real.one_half * Real.one_half * Real.one_half)) =
        one_third * (Real.one_half * Real.one_half)
    integral(centered_square_unit, Real.0, Real.1) = one_third * (Real.one_half * Real.one_half)
}

/// Twelve, as three times four.
let twelve = three * (two * two)

/// Twelve is nonzero.
lemma twelve_ne_zero {
    twelve != Real.0
} by {
    three_pos
    Real.0 < three
    two_positive
    two > Real.0
    mul_pos_pos(two, two)
    (two * two).is_positive
    two * two > Real.0
    mul_pos_pos(three, two * two)
    (three * (two * two)).is_positive
    three * (two * two) > Real.0
    twelve = three * (two * two)
    twelve > Real.0
    lt_imp_ne(Real.0, twelve)
    Real.0 != twelve
    twelve != Real.0
}

/// Twelve times one twelfth is one.
lemma twelve_mul_one_twelfth {
    twelve * (Real.1 / twelve) = Real.1
} by {
    twelve_ne_zero
    twelve != Real.0
    div_mul_cancel_denominator(Real.1, twelve)
    (Real.1 / twelve) * twelve = Real.1
    real_mul_comm(Real.1 / twelve, twelve)
    twelve * (Real.1 / twelve) = Real.1
}

/// One third times one half squared is one twelfth.
lemma one_third_times_one_half_sq_is_twelfth {
    one_third * (Real.one_half * Real.one_half) = Real.1 / twelve
} by {
    twelve_ne_zero
    twelve != Real.0
    twelve * (one_third * (Real.one_half * Real.one_half)) = three * (two * two) * (one_third * (Real.one_half * Real.one_half))
    three * (two * two) * (one_third * (Real.one_half * Real.one_half)) = (three * one_third) * ((two * two) * (Real.one_half * Real.one_half))
    three_mul_one_third
    three * one_third = Real.1
    (two * two) * (Real.one_half * Real.one_half) = (two * Real.one_half) * (two * Real.one_half)
    one_half_mul_two
    Real.one_half * two = Real.1
    real_mul_comm(Real.one_half, two)
    two * Real.one_half = Real.1
    (two * Real.one_half) * (two * Real.one_half) = Real.1 * Real.1
    Real.1 * Real.1 = Real.1
    three * (two * two) * (one_third * (Real.one_half * Real.one_half)) = Real.1
    twelve * (one_third * (Real.one_half * Real.one_half)) = Real.1
    mul_assoc(Real.1 / twelve, twelve, one_third * (Real.one_half * Real.one_half))
    (Real.1 / twelve) * (twelve * (one_third * (Real.one_half * Real.one_half))) = ((Real.1 / twelve) * twelve) * (one_third * (Real.one_half * Real.one_half))
    (Real.1 / twelve) * Real.1 = ((Real.1 / twelve) * twelve) * (one_third * (Real.one_half * Real.one_half))
    div_mul_cancel_denominator(Real.1, twelve)
    (Real.1 / twelve) * twelve = Real.1
    ((Real.1 / twelve) * twelve) * (one_third * (Real.one_half * Real.one_half)) = Real.1 * (one_third * (Real.one_half * Real.one_half))
    Real.1 * (one_third * (Real.one_half * Real.one_half)) = one_third * (Real.one_half * Real.one_half)
    (Real.1 / twelve) * Real.1 = one_third * (Real.one_half * Real.one_half)
    (Real.1 / twelve) * Real.1 = Real.1 / twelve
    one_third * (Real.one_half * Real.one_half) = Real.1 / twelve
}

/// The integral of the centered square over [0, 1] is one twelfth.
theorem integral_centered_square_unit_twelfth {
    integral(centered_square_unit, Real.0, Real.1) = Real.1 / twelve
} by {
    integral_centered_square_unit
    integral(centered_square_unit, Real.0, Real.1) = one_third * (Real.one_half * Real.one_half)
    one_third_times_one_half_sq_is_twelfth
    one_third * (Real.one_half * Real.one_half) = Real.1 / twelve
    integral(centered_square_unit, Real.0, Real.1) = Real.1 / twelve
}

/// Three is the sum of three ones.
lemma three_eq_one_plus_one_plus_one {
    three = Real.1 + Real.1 + Real.1
} by {
    three = Real.1 + Real.1 + Real.1
}

/// The centered square is integrable on [0, 1] and its integral there is one
/// third of one half squared, written without named constants.
theorem integral_centered_square_unit_third {
    is_integrable(centered_square_unit, Real.0, Real.1) and
    integral(centered_square_unit, Real.0, Real.1) = (Real.1 / (Real.1 + Real.1 + Real.1)) * (Real.one_half * Real.one_half)
} by {
    centered_square_integrable_unit
    is_integrable(centered_square_unit, Real.0, Real.1)
    integral_centered_square_unit
    integral(centered_square_unit, Real.0, Real.1) = one_third * (Real.one_half * Real.one_half)
    one_third = Real.1 / three
    three_eq_one_plus_one_plus_one
    three = Real.1 + Real.1 + Real.1
    Real.1 / three = Real.1 / (Real.1 + Real.1 + Real.1)
    one_third = Real.1 / (Real.1 + Real.1 + Real.1)
    one_third * (Real.one_half * Real.one_half) = (Real.1 / (Real.1 + Real.1 + Real.1)) * (Real.one_half * Real.one_half)
    integral(centered_square_unit, Real.0, Real.1) = (Real.1 / (Real.1 + Real.1 + Real.1)) * (Real.one_half * Real.one_half)
    is_integrable(centered_square_unit, Real.0, Real.1) and
        integral(centered_square_unit, Real.0, Real.1) = (Real.1 / (Real.1 + Real.1 + Real.1)) * (Real.one_half * Real.one_half)
}

/// A function agreeing with an integrable function on [a, b] and bounded
/// there is itself integrable and has the same integral.
theorem integral_and_integrable_eq_on(f: Real -> Real, g: Real -> Real, a: Real, b: Real, lb: Real, ub: Real) {
    a <= b and is_integrable(g, a, b) and
    (forall(t: Real) { interval_contains(a, b, t) implies f(t) = g(t) }) and
    (forall(t: Real) { interval_contains(a, b, t) implies lb <= f(t) }) and
    (forall(t: Real) { interval_contains(a, b, t) implies f(t) <= ub })
    implies
    (is_integrable(f, a, b) and integral(f, a, b) = integral(g, a, b))
} by {
    if a <= b and is_integrable(g, a, b) and
       (forall(t: Real) { interval_contains(a, b, t) implies f(t) = g(t) }) and
       (forall(t: Real) { interval_contains(a, b, t) implies lb <= f(t) }) and
       (forall(t: Real) { interval_contains(a, b, t) implies f(t) <= ub }) {
        is_integrable_eq_of_pointwise_eq_on(f, g, a, b, lb, ub)
        is_integrable(f, a, b)
        integral_eq_of_pointwise_eq_on(f, g, a, b, lb, ub)
        integral(f, a, b) = integral(g, a, b)
        is_integrable(f, a, b) and integral(f, a, b) = integral(g, a, b)
    }
}
