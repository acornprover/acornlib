from order import lt_trans, lte_antisymm, lte_of_lt, not_lt_imp_gte
from real.log import Real, exp_add, exp_log_or_zero, exp_pos, log_some_of_pos_exists
from real.harmonic import real_one_div_pos
from real.real_base import lt_add_pos
from real.real_field import mul_div_cancel
from real.real_ring import mul_neg_left, mul_neg_right

numerals Real

/// Three as a real number.
let three = Real.1 + Real.1 + Real.1

/// One third (1/3).
let one_third = Real.1 / three

/// Three is positive.
theorem three_positive {
    three > Real.0
} by {
    three = Real.1 + Real.1 + Real.1
    Real.1 > Real.0
    Real.1.is_positive
    lt_add_pos(Real.1, Real.1)
    Real.1 < Real.1 + Real.1
    lt_trans(Real.0, Real.1, Real.1 + Real.1)
    Real.0 < Real.1 + Real.1
    (Real.1 + Real.1).is_positive
    lt_add_pos(Real.1 + Real.1, Real.1)
    Real.1 + Real.1 < Real.1 + Real.1 + Real.1
    lt_trans(Real.0, Real.1 + Real.1, Real.1 + Real.1 + Real.1)
    Real.0 < Real.1 + Real.1 + Real.1
}

/// One third is positive.
theorem one_third_positive {
    one_third > Real.0
} by {
    three_positive
    real_one_div_pos(three)
    one_third > Real.0
}

/// Three copies of one third sum to one.
theorem one_third_add_three {
    one_third + one_third + one_third = Real.1
} by {
    three_positive
    three != Real.0
    mul_div_cancel(Real.1, three)
    three * (Real.1 / three) = Real.1
    one_third = Real.1 / three
    three * one_third = Real.1
    three = Real.1 + Real.1 + Real.1
    three * one_third = (Real.1 + Real.1 + Real.1) * one_third
    (Real.1 + Real.1 + Real.1) * one_third =
        Real.1 * one_third + Real.1 * one_third + Real.1 * one_third
    Real.1 * one_third = one_third
    three * one_third = one_third + one_third + one_third
    one_third + one_third + one_third = Real.1
}

/// The real cube root, with the negative branch defined by odd symmetry.
attributes Real {
    /// The real cube root, with the negative branch defined by odd symmetry.
    define cbrt(self) -> Real {
        if self < Real.0 {
            -(one_third * (-self).log.get_or_else(Real.0)).exp
        } else {
            if self > Real.0 {
                (one_third * self.log.get_or_else(Real.0)).exp
            } else {
                Real.0
            }
        }
    }
}

/// The cube root of zero is zero.
theorem cbrt_zero {
    (Real.0).cbrt = Real.0
} by {
    not Real.0 < Real.0
    not Real.0 > Real.0
    (Real.0).cbrt = Real.0
}

/// The cube root of a positive real is positive.
theorem cbrt_pos(x: Real) {
    x > Real.0 implies x.cbrt > Real.0
} by {
    not x < Real.0
    x.cbrt = (one_third * x.log.get_or_else(Real.0)).exp
    exp_pos(one_third * x.log.get_or_else(Real.0))
    x.cbrt > Real.0
}

/// The cube root of a negative real is negative.
theorem cbrt_neg(x: Real) {
    x < Real.0 implies x.cbrt < Real.0
} by {
    lte_of_lt(x, Real.0)
    x <= Real.0
    not x > Real.0
    -x > Real.0
    x.cbrt = -(one_third * (-x).log.get_or_else(Real.0)).exp
    exp_pos(one_third * (-x).log.get_or_else(Real.0))
    (one_third * (-x).log.get_or_else(Real.0)).exp > Real.0
    -(one_third * (-x).log.get_or_else(Real.0)).exp < Real.0
    x.cbrt < Real.0
}

/// Positive-base one-third powers cube back to the base.
theorem one_third_exp_cube_pos(x: Real) {
    x > Real.0 implies
    (one_third * x.log.get_or_else(Real.0)).exp * (one_third * x.log.get_or_else(Real.0)).exp * (one_third * x.log.get_or_else(Real.0)).exp = x
} by {
    let t = one_third * x.log.get_or_else(Real.0)
    exp_add(t, t)
    (t + t).exp = t.exp * t.exp
    exp_add(t + t, t)
    (t + t + t).exp = (t + t).exp * t.exp
    (t + t + t).exp = t.exp * t.exp * t.exp
    t + t + t = one_third * x.log.get_or_else(Real.0) + one_third * x.log.get_or_else(Real.0) + one_third * x.log.get_or_else(Real.0)
    one_third * x.log.get_or_else(Real.0) + one_third * x.log.get_or_else(Real.0) + one_third * x.log.get_or_else(Real.0) =
        (one_third + one_third + one_third) * x.log.get_or_else(Real.0)
    one_third_add_three
    one_third + one_third + one_third = Real.1
    (one_third + one_third + one_third) * x.log.get_or_else(Real.0) = Real.1 * x.log.get_or_else(Real.0)
    Real.1 * x.log.get_or_else(Real.0) = x.log.get_or_else(Real.0)
    t + t + t = x.log.get_or_else(Real.0)
    (t + t + t).exp = (x.log.get_or_else(Real.0)).exp
    log_some_of_pos_exists(x)
    exists(y: Real) { x.log = Option.some(y) }
    let y: Real satisfy {
        x.log = Option.some(y)
    }
    option_get_or_else_some[Real](y, Real.0)
    option_get_or_else(Option.some(y), Real.0) = y
    x.log.get_or_else(Real.0) = y
    exp_log_or_zero(x, y)
    y.exp = x
    y.exp = x
    t.exp * t.exp * t.exp = x
    (one_third * y).exp * (one_third * y).exp * (one_third * y).exp = x
}

/// The cube of the cube root of any real number is the original number.
theorem cbrt_mul_self_mul_self(x: Real) {
    x.cbrt * x.cbrt * x.cbrt = x
} by {
    if x > Real.0 {
        not x < Real.0
        x.cbrt = (one_third * x.log.get_or_else(Real.0)).exp
        (one_third * x.log.get_or_else(Real.0)).exp = x.cbrt
        one_third_exp_cube_pos(x)
        (one_third * x.log.get_or_else(Real.0)).exp * (one_third * x.log.get_or_else(Real.0)).exp * (one_third * x.log.get_or_else(Real.0)).exp = x
        x.cbrt * x.cbrt * x.cbrt = x
    } else {
        if x < Real.0 {
            lte_of_lt(x, Real.0)
            x <= Real.0
            not x > Real.0
            -x > Real.0
            x.cbrt = -(one_third * (-x).log.get_or_else(Real.0)).exp
            one_third_exp_cube_pos(-x)
            (one_third * (-x).log.get_or_else(Real.0)).exp * (one_third * (-x).log.get_or_else(Real.0)).exp * (one_third * (-x).log.get_or_else(Real.0)).exp = -x
            let y = (one_third * (-x).log.get_or_else(Real.0)).exp
            mul_neg_left(y, -y)
            mul_neg_right(y, y)
            (-y) * (-y) = y * y
            mul_neg_right(y * y, y)
            (-y) * (-y) * (-y) = -(y * y * y)
            x.cbrt * x.cbrt * x.cbrt = -(-x)
            -(-x) = x
            x.cbrt * x.cbrt * x.cbrt = x
        } else {
            not_lt_imp_gte(x, Real.0)
            x >= Real.0
            Real.0 <= x
            if not x <= Real.0 {
                x > Real.0
                false
            }
            x <= Real.0
            lte_antisymm(Real.0, x)
            Real.0 = x
            x = Real.0
            cbrt_zero
            x.cbrt = Real.0
            x.cbrt * x.cbrt * x.cbrt = Real.0
            x.cbrt * x.cbrt * x.cbrt = x
        }
    }
}
