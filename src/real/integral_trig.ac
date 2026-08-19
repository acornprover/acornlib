/// Integrals of sine and cosine over closed intervals.
///
/// The main results, integral_sin and integral_cos, state that the Darboux
/// integral of Real.sin over [a, b] equals a.cos - b.cos and the integral of Real.cos
/// over [a, b] equals b.sin - a.sin.  The route is the fundamental theorem
/// of calculus, following integral_exp.ac:
///   - the derivative of cosine is negative sine (cos_is_derivative_fn), built
///     from Real.sin' = Real.cos by the chain rule and the identity x.cos = (x + pi/2).sin,
///   - sine and cosine are bounded in absolute value by one, from the
///     Pythagorean identity (sin_sq_add_cos_sq),
///   - sine and cosine are one-Lipschitz: |x.sin - y.sin| <= |x - y| and the
///     same for cosine, by the mean value theorem,
///   - the mean-value-theorem sandwich of the Darboux sums of f between the
///     increments of its antiderivative (lower_sum_le_g_diff and
///     g_diff_le_upper_sum from integral_exp.ac) gives the endpoint values,
///   - integrability of Real.sin and Real.cos is established by comparing upper and lower
///     Darboux sums over the uniform partitions of [a, b], whose difference is
///     at most ((b - a) * (b - a)) / (n + 1) by the Lipschitz bound, and so can
///     be made arbitrarily small (fn_integrable).

from nat import Nat, from_nat, from_nat_add, lt_imp_lte_suc, pow_one
from rat import Rat
from order import lte_antisymm, lte_trans, lt_trans, lt_imp_lte, lt_imp_ne, lt_imp_ne_symm, not_lt_imp_gte, lt_of_lt_of_lte, lt_of_lte_of_lt, not_lte_imp_gt, not_lt_self
from real.real_field import Real, mul_div, mul_inverse
from real.real_base import add_comm, add_assoc, neg_distrib, abs_neg, neg_neg, lte_abs, abs_gte_zero, lte_lt_trans, lte_add_right, gt_zero_imp_pos, pos_gt_zero, sub_cancels, sub_moves_sides, lte_self, neg_lt_zero, neg_zero
from real.real_ring import from_nat_is_from_rat, mul_pos_pos, lt_mul_pos_left, lte_mul_nonneg_right, mul_sub_distrib_left, mul_abs, square_nonneg, non_neg_imp_zero_lte
from real.real_seq import sub_zero_imp_eq
from real.real_series import pow_nonneg, abs_pow, const_seq
from real.harmonic import from_nat_suc_pos_real, rat_from_nat_lte_of_nat_lte
from real.exp import pow_suc, two, two_positive
from real.trig import sin_zero, cos_zero
from real.trig_identities import sin_add, cos_add, sin_sq_add_cos_sq
from real.pi import pi_over_two, pi, cos_pi_over_two_zero, sin_pi_over_two_one, cos_pi_neg_one, sin_pi_zero, sin_continuous, cos_continuous, pi_pos, pi_over_two_pos
from real.derivative_trig import sin_has_derivative_at, sin_is_derivative_fn, abs_of_nonneg
from real.derivative_basic import has_derivative_at, has_derivative_at_unique, sub_ne_zero_of_ne, identity_has_derivative_at, constant_has_derivative_at
from real.derivative_rules import derivative_pointwise_add, derivative_pointwise_neg
from real.derivative_chain import derivative_compose
from real.derivative_continuity import div_mul_cancel_denominator
from real.mean_value import mean_value_theorem, secant_slope, lte_imp_neg_lte_neg
from real.calculus_api import is_derivative_fn, is_derivative_fn_at, is_derivative_fn_iff
from real.continuity_base import continuous, continuous_at
from real.continuity_pointwise import continuous_pointwise_neg
from data.basic.function_algebra import pointwise_add, pointwise_neg, pointwise_mul
from data.basic.functions import identity_fn, compose, function_eq_transport_predicate
from real.supremum import completeness, is_nonempty, has_upper_bound, is_set_supremum, is_set_upper_bound, is_set_infimum, is_set_lower_bound, set_member_le_supremum, set_supremum_le_upper_bound, set_upper_bound_contains_le, function_image_contains, negate_set, negate_set_contains
from real.integral import integral, is_integrable, interval_contains, interval_set, interval_image, interval_inf, interval_sup, interval_inf_spec, interval_sup_spec, lower_sum, upper_sum, lower_sum_set, upper_sum_set, lower_sum_contains, upper_sum_contains, partition_step_lower, partition_step_upper, is_partition, partition_start, partition_end, partition_mono, partition_point_in_interval, partition_monotone, diff_step, telescope, partial_lte, image_lower_bound, interval_set_contains_left, interval_set_contains_right, interval_contains_left, interval_contains_right, interval_contains_mono, sub_nonneg, integral_spec, set_supremum_unique, set_infimum_unique, sup_le_of_upper_bound, trivial_partition, trivial_partition_is_partition, neg_lte_flip, set_infimum_is_lower_bound, set_lower_bound_contains_le, set_lower_bound_le_infimum, has_lower_bound
from real.integral import neg_upper_bound_of_lower, negate_set_nonempty, inf_of_neg_sup
from real.double_sum import partial_sub_seq, sub_seq
from list import partial, partial_pointwise_eq, partial_scalar_mul
from algebra.semigroup import mul_fn
from algebra.add_ordered_group import add_le_add, add_le_add_right
from ordered_field import mul_le_mul_of_nonneg_right, mul_le_mul_of_nonneg_left
from data.basic.set import Set, maps_into_set_image
from real.am_gm import partial_const
from real.integral_exp import ftc2_general, lower_sum_le_g_diff, g_diff_le_upper_sum, image_upper_bound, uniform_partition, uniform_partition_start, uniform_partition_end, uniform_partition_monotone, uniform_partition_is_partition, uniform_partition_width, div_nonneg_pos_denom, from_nat_lte_mono, mul_frac_right, nonneg_frac_all_n_imp_zero, add_sub_cancel_shift, sub_add_cancel_shift, add_one_mul_sub

numerals Real
numerals Nat

// ---------------------------------------------------------------------------
// Pointwise bounds of sine and cosine
// ---------------------------------------------------------------------------
//
// From x.sin^2 + x.cos^2 = 1 and the nonnegativity of squares, both
// x.sin^2 and x.cos^2 are at most one, so sine and cosine lie in [-1, 1].

/// A nonnegative real whose square is at most one is at most one.
theorem nonneg_sq_le_one_imp_le_one(x: Real) {
    x >= Real.0 and x.pow(Nat.2) <= Real.1 implies x <= Real.1
} by {
    if x >= Real.0 and x.pow(Nat.2) <= Real.1 {
        if Real.1 < x {
            pow_suc(x, Nat.1)
            x.pow(Nat.1.suc) = x * x.pow(Nat.1)
            pow_one[Real](x)
            x.pow(Nat.1) = x
            Nat.1.suc = Nat.2
            x.pow(Nat.2) = x * x
            Real.0 < Real.1
            lt_trans[Real](Real.0, Real.1, x)
            Real.0 < x
            gt_zero_imp_pos(x)
            x.is_positive
            lt_mul_pos_left(Real.1, x, x)
            x * Real.1 < x * x
            x * Real.1 = x
            x < x * x
            lt_trans[Real](Real.1, x, x * x)
            Real.1 < x * x
            x.pow(Nat.2) <= Real.1
            lt_of_lte_of_lt(x.pow(Nat.2), Real.1, x.pow(Nat.2))
            x.pow(Nat.2) < x.pow(Nat.2)
            not_lt_self(x.pow(Nat.2))
            false
        }
        not Real.1 < x
        not_lt_imp_gte[Real](Real.1, x)
        x <= Real.1
    }
}

/// A real whose square is at most one has absolute value at most one.
theorem abs_le_one_of_sq_le_one(x: Real) {
    x.pow(Nat.2) <= Real.1 implies x.abs <= Real.1
} by {
    if x.pow(Nat.2) <= Real.1 {
        abs_gte_zero(x)
        Real.0 <= x.abs
        abs_pow(x, Nat.2)
        x.pow(Nat.2).abs = x.abs.pow(Nat.2)
        pow_suc(x, Nat.1)
        x.pow(Nat.1.suc) = x * x.pow(Nat.1)
        pow_one[Real](x)
        x.pow(Nat.1) = x
        Nat.1.suc = Nat.2
        x.pow(Nat.2) = x * x
        square_nonneg(x)
        x * x >= Real.0
        x.pow(Nat.2) >= Real.0
        abs_of_nonneg(x.pow(Nat.2))
        x.pow(Nat.2).abs = x.pow(Nat.2)
        x.abs.pow(Nat.2) = x.pow(Nat.2)
        x.abs.pow(Nat.2) <= Real.1
        nonneg_sq_le_one_imp_le_one(x.abs)
        x.abs <= Real.1
    }
}

/// The square of any real is nonnegative.
theorem sq_pow_nonneg(x: Real) {
    Real.0 <= x.pow(Nat.2)
} by {
    pow_suc(x, Nat.1)
    x.pow(Nat.1.suc) = x * x.pow(Nat.1)
    pow_one[Real](x)
    x.pow(Nat.1) = x
    Nat.1.suc = Nat.2
    x.pow(Nat.2) = x * x
    square_nonneg(x)
    x * x >= Real.0
    Real.0 <= x.pow(Nat.2)
}

/// The absolute value of sine is at most one.
theorem sin_abs_le_one(x: Real) {
    x.sin.abs <= Real.1
} by {
    sin_sq_add_cos_sq(x)
    x.sin.pow(Nat.2) + x.cos.pow(Nat.2) = Real.1
    sub_moves_sides(x.sin.pow(Nat.2), x.cos.pow(Nat.2), Real.1)
    x.sin.pow(Nat.2) = Real.1 - x.cos.pow(Nat.2)
    sq_pow_nonneg(x.cos)
    Real.0 <= x.cos.pow(Nat.2)
    lte_imp_neg_lte_neg(Real.0, x.cos.pow(Nat.2))
    -x.cos.pow(Nat.2) <= -Real.0
    neg_zero
    -Real.0 = Real.0
    -x.cos.pow(Nat.2) <= Real.0
    lte_self(Real.1)
    Real.1 <= Real.1
    add_le_add[Real](Real.1, Real.1, -x.cos.pow(Nat.2), Real.0)
    Real.1 + -x.cos.pow(Nat.2) <= Real.1 + Real.0
    Real.1 - x.cos.pow(Nat.2) = Real.1 + -x.cos.pow(Nat.2)
    Real.1 - x.cos.pow(Nat.2) <= Real.1 + Real.0
    Real.1 + Real.0 = Real.1
    Real.1 - x.cos.pow(Nat.2) <= Real.1
    x.sin.pow(Nat.2) <= Real.1
    abs_le_one_of_sq_le_one(x.sin)
    x.sin.abs <= Real.1
}

/// The absolute value of cosine is at most one.
theorem cos_abs_le_one(x: Real) {
    x.cos.abs <= Real.1
} by {
    sin_sq_add_cos_sq(x)
    x.sin.pow(Nat.2) + x.cos.pow(Nat.2) = Real.1
    add_comm(x.sin.pow(Nat.2), x.cos.pow(Nat.2))
    x.cos.pow(Nat.2) + x.sin.pow(Nat.2) = Real.1
    sub_moves_sides(x.cos.pow(Nat.2), x.sin.pow(Nat.2), Real.1)
    x.cos.pow(Nat.2) = Real.1 - x.sin.pow(Nat.2)
    sq_pow_nonneg(x.sin)
    Real.0 <= x.sin.pow(Nat.2)
    lte_imp_neg_lte_neg(Real.0, x.sin.pow(Nat.2))
    -x.sin.pow(Nat.2) <= -Real.0
    neg_zero
    -Real.0 = Real.0
    -x.sin.pow(Nat.2) <= Real.0
    lte_self(Real.1)
    Real.1 <= Real.1
    add_le_add[Real](Real.1, Real.1, -x.sin.pow(Nat.2), Real.0)
    Real.1 + -x.sin.pow(Nat.2) <= Real.1 + Real.0
    Real.1 - x.sin.pow(Nat.2) = Real.1 + -x.sin.pow(Nat.2)
    Real.1 - x.sin.pow(Nat.2) <= Real.1 + Real.0
    Real.1 + Real.0 = Real.1
    Real.1 - x.sin.pow(Nat.2) <= Real.1
    x.cos.pow(Nat.2) <= Real.1
    abs_le_one_of_sq_le_one(x.cos)
    x.cos.abs <= Real.1
}

/// Sine is at most one.
theorem sin_le_one(x: Real) {
    x.sin <= Real.1
} by {
    sin_abs_le_one(x)
    x.sin.abs <= Real.1
    lte_abs(x.sin)
    x.sin <= x.sin.abs
    lte_trans[Real](x.sin, x.sin.abs, Real.1)
    x.sin <= Real.1
}

/// Sine is at least negative one.
theorem sin_ge_neg_one(x: Real) {
    -Real.1 <= x.sin
} by {
    lte_abs(-x.sin)
    -x.sin <= (-x.sin).abs
    abs_neg(x.sin)
    (-x.sin).abs = x.sin.abs
    -x.sin <= x.sin.abs
    sin_abs_le_one(x)
    x.sin.abs <= Real.1
    lte_trans[Real](-x.sin, x.sin.abs, Real.1)
    -x.sin <= Real.1
    lte_imp_neg_lte_neg(-x.sin, Real.1)
    -Real.1 <= -(-x.sin)
    neg_neg(x.sin)
    -(-x.sin) = x.sin
    -Real.1 <= x.sin
}

/// Cosine is at most one.
theorem cos_le_one(x: Real) {
    x.cos <= Real.1
} by {
    cos_abs_le_one(x)
    x.cos.abs <= Real.1
    lte_abs(x.cos)
    x.cos <= x.cos.abs
    lte_trans[Real](x.cos, x.cos.abs, Real.1)
    x.cos <= Real.1
}

/// Cosine is at least negative one.
theorem cos_ge_neg_one(x: Real) {
    -Real.1 <= x.cos
} by {
    lte_abs(-x.cos)
    -x.cos <= (-x.cos).abs
    abs_neg(x.cos)
    (-x.cos).abs = x.cos.abs
    -x.cos <= x.cos.abs
    cos_abs_le_one(x)
    x.cos.abs <= Real.1
    lte_trans[Real](-x.cos, x.cos.abs, Real.1)
    -x.cos <= Real.1
    lte_imp_neg_lte_neg(-x.cos, Real.1)
    -Real.1 <= -(-x.cos)
    neg_neg(x.cos)
    -(-x.cos) = x.cos
    -Real.1 <= x.cos
}

// ---------------------------------------------------------------------------
// The derivative of cosine
// ---------------------------------------------------------------------------
//
// x.cos = (x + pi/2).sin, so by the chain rule applied to the shift
// x ↦ x + pi/2 composed with sine, the derivative of cosine at x is
// (x + pi/2).cos = -x.sin.

/// The sine of x plus pi over two is cosine of x.
theorem sin_add_pi_over_two(x: Real) {
    (x + pi_over_two).sin = x.cos
} by {
    sin_add(x, pi_over_two)
    (x + pi_over_two).sin = x.sin * pi_over_two.cos + x.cos * pi_over_two.sin
    cos_pi_over_two_zero
    pi_over_two.cos = Real.0
    sin_pi_over_two_one
    pi_over_two.sin = Real.1
    x.sin * pi_over_two.cos + x.cos * pi_over_two.sin = x.sin * Real.0 + x.cos * Real.1
    x.sin * Real.0 = Real.0
    x.cos * Real.1 = x.cos
    x.sin * Real.0 + x.cos * Real.1 = Real.0 + x.cos
    Real.0 + x.cos = x.cos
    (x + pi_over_two).sin = x.cos
}

/// The cosine of x plus pi over two is negative sine of x.
theorem cos_add_pi_over_two(x: Real) {
    (x + pi_over_two).cos = -x.sin
} by {
    cos_add(x, pi_over_two)
    (x + pi_over_two).cos = x.cos * pi_over_two.cos - x.sin * pi_over_two.sin
    cos_pi_over_two_zero
    pi_over_two.cos = Real.0
    sin_pi_over_two_one
    pi_over_two.sin = Real.1
    x.cos * pi_over_two.cos - x.sin * pi_over_two.sin = x.cos * Real.0 - x.sin * Real.1
    x.cos * Real.0 = Real.0
    x.sin * Real.1 = x.sin
    x.cos * Real.0 - x.sin * Real.1 = Real.0 - x.sin
    Real.0 - x.sin = -x.sin
    (x + pi_over_two).cos = -x.sin
}

/// The cosine function is everywhere differentiable with derivative negative sine.
theorem cos_is_derivative_fn {
    is_derivative_fn(Real.cos, pointwise_neg(Real.sin))
} by {
    forall(y: Real) {
        compose(Real.sin, pointwise_add(identity_fn[Real], constant[Real, Real](pi_over_two)), y) =
            (pointwise_add(identity_fn[Real], constant[Real, Real](pi_over_two), y)).sin
        pointwise_add(identity_fn[Real], constant[Real, Real](pi_over_two), y) =
            identity_fn[Real](y) + constant[Real, Real](pi_over_two, y)
        identity_fn[Real](y) = y
        constant[Real, Real](pi_over_two, y) = pi_over_two
        pointwise_add(identity_fn[Real], constant[Real, Real](pi_over_two), y) = y + pi_over_two
        compose(Real.sin, pointwise_add(identity_fn[Real], constant[Real, Real](pi_over_two)), y) =
            (y + pi_over_two).sin
        sin_add_pi_over_two(y)
        (y + pi_over_two).sin = y.cos
        compose(Real.sin, pointwise_add(identity_fn[Real], constant[Real, Real](pi_over_two)), y) = y.cos
    }
    compose(Real.sin, pointwise_add(identity_fn[Real], constant[Real, Real](pi_over_two))) = Real.cos
    forall(x: Real) {
        identity_has_derivative_at(x)
        has_derivative_at(identity_fn[Real], x, Real.1)
        constant_has_derivative_at(pi_over_two, x)
        has_derivative_at(constant[Real, Real](pi_over_two), x, Real.0)
        derivative_pointwise_add(identity_fn[Real], constant[Real, Real](pi_over_two), x, Real.1, Real.0)
        has_derivative_at(pointwise_add(identity_fn[Real], constant[Real, Real](pi_over_two)), x,
            Real.1 + Real.0)
        Real.1 + Real.0 = Real.1
        has_derivative_at(pointwise_add(identity_fn[Real], constant[Real, Real](pi_over_two)), x, Real.1)
        pointwise_add(identity_fn[Real], constant[Real, Real](pi_over_two), x) =
            identity_fn[Real](x) + constant[Real, Real](pi_over_two, x)
        identity_fn[Real](x) = x
        constant[Real, Real](pi_over_two, x) = pi_over_two
        pointwise_add(identity_fn[Real], constant[Real, Real](pi_over_two), x) = x + pi_over_two
        sin_has_derivative_at(x + pi_over_two)
        has_derivative_at(Real.sin, x + pi_over_two, (x + pi_over_two).cos)
        derivative_compose(pointwise_add(identity_fn[Real], constant[Real, Real](pi_over_two)), Real.sin,
            x, Real.1, (x + pi_over_two).cos)
        has_derivative_at(compose(Real.sin, pointwise_add(identity_fn[Real], constant[Real, Real](pi_over_two))),
            x, (x + pi_over_two).cos * Real.1)
        (x + pi_over_two).cos * Real.1 = (x + pi_over_two).cos
        has_derivative_at(compose(Real.sin, pointwise_add(identity_fn[Real], constant[Real, Real](pi_over_two))),
            x, (x + pi_over_two).cos)
        cos_add_pi_over_two(x)
        (x + pi_over_two).cos = -x.sin
        pointwise_neg(Real.sin, x) = -x.sin
        has_derivative_at(compose(Real.sin, pointwise_add(identity_fn[Real], constant[Real, Real](pi_over_two))),
            x, pointwise_neg(Real.sin, x))
        define pred(h: Real -> Real) -> Bool {
            has_derivative_at(h, x, pointwise_neg(Real.sin, x))
        }
        function_eq_transport_predicate(pred,
            compose(Real.sin, pointwise_add(identity_fn[Real], constant[Real, Real](pi_over_two))), Real.cos)
        has_derivative_at(Real.cos, x, pointwise_neg(Real.sin, x))
    }
    is_derivative_fn_iff(Real.cos, pointwise_neg(Real.sin))
    is_derivative_fn(Real.cos, pointwise_neg(Real.sin))
}

/// The cosine has derivative negative sine at every point.
theorem cos_has_derivative_at(x0: Real) {
    has_derivative_at(Real.cos, x0, -x0.sin)
} by {
    cos_is_derivative_fn
    is_derivative_fn_at(Real.cos, pointwise_neg(Real.sin), x0)
    has_derivative_at(Real.cos, x0, pointwise_neg(Real.sin, x0))
    pointwise_neg(Real.sin, x0) = -x0.sin
    has_derivative_at(Real.cos, x0, -x0.sin)
}

/// The pointwise negation of cosine has derivative sine everywhere.
theorem neg_cos_is_derivative_fn {
    is_derivative_fn(pointwise_neg(Real.cos), Real.sin)
} by {
    forall(x: Real) {
        cos_has_derivative_at(x)
        has_derivative_at(Real.cos, x, -x.sin)
        derivative_pointwise_neg(Real.cos, x, -x.sin)
        has_derivative_at(pointwise_neg(Real.cos), x, -(-x.sin))
        neg_neg(x.sin)
        -(-x.sin) = x.sin
        has_derivative_at(pointwise_neg(Real.cos), x, x.sin)
    }
    is_derivative_fn_iff(pointwise_neg(Real.cos), Real.sin)
    is_derivative_fn(pointwise_neg(Real.cos), Real.sin)
}

/// The pointwise negation of cosine is continuous everywhere.
theorem neg_cos_continuous {
    continuous(pointwise_neg(Real.cos))
} by {
    cos_continuous
    continuous(Real.cos)
    continuous_pointwise_neg(Real.cos)
    continuous(pointwise_neg(Real.cos))
}

/// On [a, b] sine is bounded below by negative one.
theorem sin_lower_bound_on(a: Real, b: Real) {
    a <= b implies forall(t: Real) { interval_contains(a, b, t) implies -Real.1 <= t.sin }
} by {
    if a <= b {
        forall(t: Real) {
            if interval_contains(a, b, t) {
                sin_ge_neg_one(t)
                -Real.1 <= t.sin
            }
        }
    }
}

/// On [a, b] sine is bounded above by one.
theorem sin_upper_bound_on(a: Real, b: Real) {
    a <= b implies forall(t: Real) { interval_contains(a, b, t) implies t.sin <= Real.1 }
} by {
    if a <= b {
        forall(t: Real) {
            if interval_contains(a, b, t) {
                sin_le_one(t)
                t.sin <= Real.1
            }
        }
    }
}

/// On [a, b] cosine is bounded below by negative one.
theorem cos_lower_bound_on(a: Real, b: Real) {
    a <= b implies forall(t: Real) { interval_contains(a, b, t) implies -Real.1 <= t.cos }
} by {
    if a <= b {
        forall(t: Real) {
            if interval_contains(a, b, t) {
                cos_ge_neg_one(t)
                -Real.1 <= t.cos
            }
        }
    }
}

/// On [a, b] cosine is bounded above by one.
theorem cos_upper_bound_on(a: Real, b: Real) {
    a <= b implies forall(t: Real) { interval_contains(a, b, t) implies t.cos <= Real.1 }
} by {
    if a <= b {
        forall(t: Real) {
            if interval_contains(a, b, t) {
                cos_le_one(t)
                t.cos <= Real.1
            }
        }
    }
}

// ---------------------------------------------------------------------------
// Sine and cosine are one-Lipschitz
// ---------------------------------------------------------------------------
//
// By the mean value theorem on [x, y], y.sin - x.sin = c.cos * (y - x) for
// some c between x and y; since |c.cos| <= 1, |y.sin - x.sin| <= |y - x|.
// The argument for cosine is identical with derivative -Real.sin.

/// If f has global derivative df with |df| <= m everywhere, then f is
/// m-Lipschitz on the increasing interval [x, y].
theorem lipschitz_lt_of_bounded_derivative(f: Real -> Real, df: Real -> Real, m: Real, x: Real, y: Real) {
    x < y and continuous(f) and is_derivative_fn(f, df) and
    (forall(z: Real) { df(z).abs <= m }) implies
    (f(x) - f(y)).abs <= m * (y - x)
} by {
    if x < y and continuous(f) and is_derivative_fn(f, df) and
       (forall(z: Real) { df(z).abs <= m }) {
        mean_value_theorem(f, df, x, y)
        let c: Real satisfy {
            x < c and c < y and has_derivative_at(f, c, secant_slope(f, x, y))
        }
        x < c and c < y
        has_derivative_at(f, c, secant_slope(f, x, y))
        is_derivative_fn_at(f, df, c)
        has_derivative_at(f, c, df(c))
        has_derivative_at_unique(f, c, secant_slope(f, x, y), df(c))
        secant_slope(f, x, y) = df(c)
        secant_slope(f, x, y) = (f(y) - f(x)) / (y - x)
        df(c) = (f(y) - f(x)) / (y - x)
        lt_imp_ne_symm(x, y)
        y != x
        sub_ne_zero_of_ne(y, x)
        y - x != Real.0
        div_mul_cancel_denominator(f(y) - f(x), y - x)
        ((f(y) - f(x)) / (y - x)) * (y - x) = f(y) - f(x)
        df(c) * (y - x) = f(y) - f(x)
        mul_abs(df(c), y - x)
        (df(c) * (y - x)).abs = df(c).abs * (y - x).abs
        lt_imp_lte(x, y)
        x <= y
        sub_nonneg(x, y)
        Real.0 <= y - x
        abs_of_nonneg(y - x)
        (y - x).abs = y - x
        (df(c) * (y - x)).abs = df(c).abs * (y - x)
        (f(y) - f(x)).abs = (df(c) * (y - x)).abs
        (f(y) - f(x)).abs = df(c).abs * (y - x)
        forall(z: Real) {
            df(z).abs <= m
        }
        df(c).abs <= m
        mul_le_mul_of_nonneg_right(df(c).abs, m, y - x)
        df(c).abs * (y - x) <= m * (y - x)
        (f(y) - f(x)).abs <= m * (y - x)
        abs_neg(f(x) - f(y))
        (-(f(x) - f(y))).abs = (f(x) - f(y)).abs
        -(f(x) - f(y)) = f(y) - f(x)
        (f(x) - f(y)).abs = (f(y) - f(x)).abs
        (f(x) - f(y)).abs <= m * (y - x)
    }
}

/// Sine is one-Lipschitz on increasing intervals.
theorem sin_lipschitz_lt(x: Real, y: Real) {
    x < y implies (x.sin - y.sin).abs <= (x - y).abs
} by {
    if x < y {
        sin_continuous
        continuous(Real.sin)
        sin_is_derivative_fn
        is_derivative_fn(Real.sin, Real.cos)
        forall(z: Real) {
            cos_abs_le_one(z)
            z.cos.abs <= Real.1
        }
        lipschitz_lt_of_bounded_derivative(Real.sin, Real.cos, Real.1, x, y)
        (x.sin - y.sin).abs <= Real.1 * (y - x)
        Real.1 * (y - x) = y - x
        (x.sin - y.sin).abs <= y - x
        lt_imp_lte(x, y)
        x <= y
        sub_nonneg(x, y)
        Real.0 <= y - x
        abs_of_nonneg(y - x)
        (y - x).abs = y - x
        abs_neg(x - y)
        (-(x - y)).abs = (x - y).abs
        -(x - y) = y - x
        (x - y).abs = (y - x).abs
        (x - y).abs = y - x
        (x.sin - y.sin).abs <= (x - y).abs
    }
}

/// Cosine is one-Lipschitz on increasing intervals.
theorem cos_lipschitz_lt(x: Real, y: Real) {
    x < y implies (x.cos - y.cos).abs <= (x - y).abs
} by {
    if x < y {
        cos_continuous
        continuous(Real.cos)
        cos_is_derivative_fn
        is_derivative_fn(Real.cos, pointwise_neg(Real.sin))
        forall(z: Real) {
            pointwise_neg(Real.sin, z) = -z.sin
            abs_neg(z.sin)
            (-z.sin).abs = z.sin.abs
            sin_abs_le_one(z)
            z.sin.abs <= Real.1
            pointwise_neg(Real.sin, z).abs <= Real.1
        }
        lipschitz_lt_of_bounded_derivative(Real.cos, pointwise_neg(Real.sin), Real.1, x, y)
        (x.cos - y.cos).abs <= Real.1 * (y - x)
        Real.1 * (y - x) = y - x
        (x.cos - y.cos).abs <= y - x
        lt_imp_lte(x, y)
        x <= y
        sub_nonneg(x, y)
        Real.0 <= y - x
        abs_of_nonneg(y - x)
        (y - x).abs = y - x
        abs_neg(x - y)
        (-(x - y)).abs = (x - y).abs
        -(x - y) = y - x
        (x - y).abs = (y - x).abs
        (x - y).abs = y - x
        (x.cos - y.cos).abs <= (x - y).abs
    }
}

/// If x is not less than y and they are distinct, then y is less than x.
theorem not_lt_and_ne_imp_gt(x: Real, y: Real) {
    not x < y and x != y implies y < x
} by {
    if not x < y and x != y {
        not_lt_imp_gte[Real](x, y)
        y <= x
        if y < x {
            y < x
        } else {
            not y < x
            not_lt_imp_gte[Real](y, x)
            x <= y
            lte_antisymm[Real](x, y)
            x = y
            x != y
            false
        }
    }
}

/// Sine is one-Lipschitz: |x.sin - y.sin| <= |x - y|.
theorem sin_lipschitz(x: Real, y: Real) {
    (x.sin - y.sin).abs <= (x - y).abs
} by {
    if x < y {
        sin_lipschitz_lt(x, y)
        (x.sin - y.sin).abs <= (x - y).abs
    } else {
        not x < y
        not_lt_imp_gte[Real](x, y)
        y <= x
        if x = y {
            x.sin - y.sin = Real.0
            x - y = Real.0
            (x.sin - y.sin).abs = Real.0.abs
            (x - y).abs = Real.0.abs
            lte_self(Real.0.abs)
            Real.0.abs <= Real.0.abs
            (x.sin - y.sin).abs <= (x - y).abs
        } else {
            not x = y
            x != y
            not_lt_and_ne_imp_gt(x, y)
            y < x
            sin_lipschitz_lt(y, x)
            (y.sin - x.sin).abs <= (y - x).abs
            abs_neg(x.sin - y.sin)
            (-(x.sin - y.sin)).abs = (x.sin - y.sin).abs
            -(x.sin - y.sin) = y.sin - x.sin
            (x.sin - y.sin).abs = (y.sin - x.sin).abs
            abs_neg(x - y)
            (-(x - y)).abs = (x - y).abs
            -(x - y) = y - x
            (x - y).abs = (y - x).abs
            (x.sin - y.sin).abs <= (x - y).abs
        }
    }
}

/// Cosine is one-Lipschitz: |x.cos - y.cos| <= |x - y|.
theorem cos_lipschitz(x: Real, y: Real) {
    (x.cos - y.cos).abs <= (x - y).abs
} by {
    if x < y {
        cos_lipschitz_lt(x, y)
        (x.cos - y.cos).abs <= (x - y).abs
    } else {
        not x < y
        not_lt_imp_gte[Real](x, y)
        y <= x
        if x = y {
            x.cos - y.cos = Real.0
            x - y = Real.0
            (x.cos - y.cos).abs = Real.0.abs
            (x - y).abs = Real.0.abs
            lte_self(Real.0.abs)
            Real.0.abs <= Real.0.abs
            (x.cos - y.cos).abs <= (x - y).abs
        } else {
            not x = y
            x != y
            not_lt_and_ne_imp_gt(x, y)
            y < x
            cos_lipschitz_lt(y, x)
            (y.cos - x.cos).abs <= (y - x).abs
            abs_neg(x.cos - y.cos)
            (-(x.cos - y.cos)).abs = (x.cos - y.cos).abs
            -(x.cos - y.cos) = y.cos - x.cos
            (x.cos - y.cos).abs = (y.cos - x.cos).abs
            abs_neg(x - y)
            (-(x - y)).abs = (x - y).abs
            -(x - y) = y - x
            (x - y).abs = (y - x).abs
            (x.cos - y.cos).abs <= (x - y).abs
        }
    }
}

// ---------------------------------------------------------------------------
// Small-angle bounds for sine
// ---------------------------------------------------------------------------
//
// Since sine is one-Lipschitz and (0).sin = 0, the absolute value of sine is
// bounded by the absolute value of its argument.  For nonnegative arguments
// this gives the classical inequality x.sin <= x.

/// The absolute value of sine is at most the absolute value of its argument.
theorem sin_abs_le_abs(x: Real) {
    x.sin.abs <= x.abs
} by {
    sin_lipschitz(x, Real.0)
    (x.sin - (Real.0).sin).abs <= (x - Real.0).abs
    sin_zero
    (Real.0).sin = Real.0
    x.sin - (Real.0).sin = x.sin
    (x.sin - (Real.0).sin).abs = x.sin.abs
    x - Real.0 = x
    (x - Real.0).abs = x.abs
    x.sin.abs <= x.abs
}

/// Sine is at most its argument on nonnegative reals.
theorem sin_le_x_nonneg(x: Real) {
    x >= Real.0 implies x.sin <= x
} by {
    if x >= Real.0 {
        sin_abs_le_abs(x)
        x.sin.abs <= x.abs
        abs_of_nonneg(x)
        x.abs = x
        x.sin.abs <= x
        lte_abs(x.sin)
        x.sin <= x.sin.abs
        lte_trans[Real](x.sin, x.sin.abs, x)
        x.sin <= x
    }
}

// ---------------------------------------------------------------------------
// Supremum minus infimum over an interval
// ---------------------------------------------------------------------------

/// A point of [x, y] is within y - x of another point of [x, y].
theorem interval_abs_diff_le_width(x: Real, y: Real, u: Real, v: Real) {
    x <= u and u <= y and x <= v and v <= y implies (u - v).abs <= y - x
} by {
    if x <= u and u <= y and x <= v and v <= y {
        lte_imp_neg_lte_neg(x, v)
        -v <= -x
        add_le_add[Real](u, y, -v, -x)
        u + -v <= y + -x
        u - v = u + -v
        y - x = y + -x
        u - v <= y - x
        lte_imp_neg_lte_neg(x, u)
        -u <= -x
        add_le_add[Real](v, y, -u, -x)
        v + -u <= y + -x
        v - u = v + -u
        v - u <= y - x
        if (u - v).is_negative {
            neg_lt_zero(u - v)
            u - v < Real.0
            (u - v).abs = -(u - v)
            -(u - v) = v - u
            (u - v).abs = v - u
            (u - v).abs <= y - x
        } else {
            not (u - v).is_negative
            non_neg_imp_zero_lte(u - v)
            Real.0 <= u - v
            abs_of_nonneg(u - v)
            (u - v).abs = u - v
            (u - v).abs <= y - x
        }
    }
}

/// If f is (y - x)-Lipschitz on [x, y] and bounded there, its supremum and
/// infimum over [x, y] differ by at most y - x.
theorem interval_sup_sub_inf_le_lipschitz(f: Real -> Real, x: Real, y: Real) {
    x <= y and is_nonempty(interval_set(x, y)) and has_upper_bound(interval_image(f, x, y)) and
    has_lower_bound(interval_image(f, x, y)) and
    (forall(u: Real, v: Real) {
        interval_contains(x, y, u) and interval_contains(x, y, v) implies (f(u) - f(v)).abs <= y - x
    }) implies
    interval_sup(f, x, y) - interval_inf(f, x, y) <= y - x
} by {
    if x <= y and is_nonempty(interval_set(x, y)) and has_upper_bound(interval_image(f, x, y)) and
       has_lower_bound(interval_image(f, x, y)) and
       (forall(u: Real, v: Real) {
           interval_contains(x, y, u) and interval_contains(x, y, v) implies (f(u) - f(v)).abs <= y - x
       }) {
        interval_sup_spec(f, x, y)
        is_set_supremum(interval_image(f, x, y), interval_sup(f, x, y))
        forall(v: Real) {
            if interval_image(f, x, y).contains(v) {
                interval_image(f, x, y).contains(v) = function_image_contains(f, interval_set(x, y), v)
                function_image_contains(f, interval_set(x, y), v)
                let t: Real satisfy {
                    interval_set(x, y).contains(t) and v = f(t)
                }
                interval_set(x, y).contains(t) = interval_contains(x, y, t)
                interval_contains(x, y, t)
                forall(w: Real) {
                    if interval_image(f, x, y).contains(w) {
                        interval_image(f, x, y).contains(w) = function_image_contains(f, interval_set(x, y), w)
                        function_image_contains(f, interval_set(x, y), w)
                        let s: Real satisfy {
                            interval_set(x, y).contains(s) and w = f(s)
                        }
                        interval_set(x, y).contains(s) = interval_contains(x, y, s)
                        interval_contains(x, y, s)
                        forall(u: Real, v0: Real) {
                            interval_contains(x, y, u) and interval_contains(x, y, v0) implies (f(u) - f(v0)).abs <= y - x
                        }
                        interval_contains(x, y, s) and interval_contains(x, y, t) implies (f(s) - f(t)).abs <= y - x
                        interval_contains(x, y, s) and interval_contains(x, y, t)
                        (f(s) - f(t)).abs <= y - x
                        lte_abs(f(s) - f(t))
                        f(s) - f(t) <= (f(s) - f(t)).abs
                        lte_trans[Real](f(s) - f(t), (f(s) - f(t)).abs, y - x)
                        f(s) - f(t) <= y - x
                        add_le_add_right[Real](f(s) - f(t), y - x, f(t))
                        f(s) - f(t) + f(t) <= y - x + f(t)
                        f(s) - f(t) + f(t) = f(s)
                        f(s) <= y - x + f(t)
                        y - x + f(t) = f(t) + (y - x)
                        f(s) <= f(t) + (y - x)
                        w = f(s)
                        v = f(t)
                        w <= v + (y - x)
                    }
                }
                is_set_upper_bound(interval_image(f, x, y), v + (y - x))
                sup_le_of_upper_bound(interval_image(f, x, y), interval_sup(f, x, y), v + (y - x))
                interval_sup(f, x, y) <= v + (y - x)
                add_le_add_right[Real](interval_sup(f, x, y), v + (y - x), -(y - x))
                interval_sup(f, x, y) + -(y - x) <= v + (y - x) + -(y - x)
                interval_sup(f, x, y) - (y - x) = interval_sup(f, x, y) + -(y - x)
                v + (y - x) + -(y - x) = v + (y - x) - (y - x)
                interval_sup(f, x, y) - (y - x) <= v + (y - x) - (y - x)
                v + (y - x) - (y - x) = v
                interval_sup(f, x, y) - (y - x) <= v
            }
        }
        is_set_lower_bound(interval_image(f, x, y), interval_sup(f, x, y) - (y - x))
        interval_inf_spec(f, x, y)
        is_set_infimum(interval_image(f, x, y), interval_inf(f, x, y))
        set_lower_bound_le_infimum(interval_image(f, x, y), interval_inf(f, x, y),
            interval_sup(f, x, y) - (y - x))
        interval_sup(f, x, y) - (y - x) <= interval_inf(f, x, y)
        add_le_add_right[Real](interval_sup(f, x, y) - (y - x), interval_inf(f, x, y), y - x)
        interval_sup(f, x, y) - (y - x) + (y - x) <= interval_inf(f, x, y) + (y - x)
        interval_sup(f, x, y) - (y - x) + (y - x) = interval_sup(f, x, y)
        interval_sup(f, x, y) <= interval_inf(f, x, y) + (y - x)
        add_le_add_right[Real](interval_sup(f, x, y), interval_inf(f, x, y) + (y - x),
            -interval_inf(f, x, y))
        interval_sup(f, x, y) + -interval_inf(f, x, y) <= interval_inf(f, x, y) + (y - x) + -interval_inf(f, x, y)
        interval_sup(f, x, y) - interval_inf(f, x, y) = interval_sup(f, x, y) + -interval_inf(f, x, y)
        interval_inf(f, x, y) + (y - x) + -interval_inf(f, x, y) = interval_inf(f, x, y) + (y - x) - interval_inf(f, x, y)
        add_comm(interval_inf(f, x, y), y - x)
        interval_inf(f, x, y) + (y - x) = (y - x) + interval_inf(f, x, y)
        interval_inf(f, x, y) + (y - x) - interval_inf(f, x, y) = (y - x) + interval_inf(f, x, y) - interval_inf(f, x, y)
        sub_cancels(y - x, interval_inf(f, x, y))
        (y - x) + interval_inf(f, x, y) - interval_inf(f, x, y) = y - x
        interval_inf(f, x, y) + (y - x) - interval_inf(f, x, y) = y - x
        interval_sup(f, x, y) - interval_inf(f, x, y) <= y - x
    }
}

/// The supremum of sine over [x, y] is within y - x of its infimum.
theorem sin_interval_sup_sub_inf_le_width(x: Real, y: Real) {
    x <= y implies interval_sup(Real.sin, x, y) - interval_inf(Real.sin, x, y) <= y - x
} by {
    if x <= y {
        interval_set_contains_left(x, y)
        interval_set(x, y).contains(x)
        exists(z: Real) {
            interval_set(x, y).contains(z)
        }
        is_nonempty(interval_set(x, y))
        forall(t: Real) {
            if interval_contains(x, y, t) {
                sin_le_one(t)
                t.sin <= Real.1
            }
        }
        image_upper_bound(Real.sin, x, y, Real.1)
        is_set_upper_bound(interval_image(Real.sin, x, y), Real.1)
        exists(b: Real) {
            is_set_upper_bound(interval_image(Real.sin, x, y), b)
        }
        has_upper_bound(interval_image(Real.sin, x, y))
        forall(t: Real) {
            if interval_contains(x, y, t) {
                sin_ge_neg_one(t)
                -Real.1 <= t.sin
            }
        }
        image_lower_bound(Real.sin, x, y, -Real.1)
        is_set_lower_bound(interval_image(Real.sin, x, y), -Real.1)
        exists(b: Real) {
            is_set_lower_bound(interval_image(Real.sin, x, y), b)
        }
        has_lower_bound(interval_image(Real.sin, x, y))
        forall(u: Real, v: Real) {
            if interval_contains(x, y, u) and interval_contains(x, y, v) {
                interval_contains_left(x, y, u)
                x <= u
                interval_contains_right(x, y, u)
                u <= y
                interval_contains_left(x, y, v)
                x <= v
                interval_contains_right(x, y, v)
                v <= y
                interval_abs_diff_le_width(x, y, u, v)
                (u - v).abs <= y - x
                sin_lipschitz(u, v)
                (u.sin - v.sin).abs <= (u - v).abs
                lte_trans[Real]((u.sin - v.sin).abs, (u - v).abs, y - x)
                (u.sin - v.sin).abs <= y - x
            }
        }
        interval_sup_sub_inf_le_lipschitz(Real.sin, x, y)
        interval_sup(Real.sin, x, y) - interval_inf(Real.sin, x, y) <= y - x
    }
}

/// The supremum of cosine over [x, y] is within y - x of its infimum.
theorem cos_interval_sup_sub_inf_le_width(x: Real, y: Real) {
    x <= y implies interval_sup(Real.cos, x, y) - interval_inf(Real.cos, x, y) <= y - x
} by {
    if x <= y {
        interval_set_contains_left(x, y)
        interval_set(x, y).contains(x)
        exists(z: Real) {
            interval_set(x, y).contains(z)
        }
        is_nonempty(interval_set(x, y))
        forall(t: Real) {
            if interval_contains(x, y, t) {
                cos_le_one(t)
                t.cos <= Real.1
            }
        }
        image_upper_bound(Real.cos, x, y, Real.1)
        is_set_upper_bound(interval_image(Real.cos, x, y), Real.1)
        exists(b: Real) {
            is_set_upper_bound(interval_image(Real.cos, x, y), b)
        }
        has_upper_bound(interval_image(Real.cos, x, y))
        forall(t: Real) {
            if interval_contains(x, y, t) {
                cos_ge_neg_one(t)
                -Real.1 <= t.cos
            }
        }
        image_lower_bound(Real.cos, x, y, -Real.1)
        is_set_lower_bound(interval_image(Real.cos, x, y), -Real.1)
        exists(b: Real) {
            is_set_lower_bound(interval_image(Real.cos, x, y), b)
        }
        has_lower_bound(interval_image(Real.cos, x, y))
        forall(u: Real, v: Real) {
            if interval_contains(x, y, u) and interval_contains(x, y, v) {
                interval_contains_left(x, y, u)
                x <= u
                interval_contains_right(x, y, u)
                u <= y
                interval_contains_left(x, y, v)
                x <= v
                interval_contains_right(x, y, v)
                v <= y
                interval_abs_diff_le_width(x, y, u, v)
                (u - v).abs <= y - x
                cos_lipschitz(u, v)
                (u.cos - v.cos).abs <= (u - v).abs
                lte_trans[Real]((u.cos - v.cos).abs, (u - v).abs, y - x)
                (u.cos - v.cos).abs <= y - x
            }
        }
        interval_sup_sub_inf_le_lipschitz(Real.cos, x, y)
        interval_sup(Real.cos, x, y) - interval_inf(Real.cos, x, y) <= y - x
    }
}

// ---------------------------------------------------------------------------
// The uniform partition bound
// ---------------------------------------------------------------------------
//
// On each subinterval of the uniform partition of [a, b], the supremum and
// infimum of a one-Lipschitz f differ by at most the mesh, so the difference
// of the upper and lower Darboux sums is at most the mesh times the total
// width, ((b - a) * (b - a)) / (n + 1).

/// The common width of the subintervals of the uniform partition of [a, b] of
/// length n + 1.
define uniform_mesh(a: Real, b: Real, n: Nat) -> Real {
    (b - a) / from_nat[Real](n.suc)
}

/// If f is one-Lipschitz and bounded by one in absolute value on [a, b], the
/// difference of its upper and lower Darboux sums over the uniform partition
/// of [a, b] into n + 1 equal parts is at most ((b - a) * (b - a)) / (n + 1).
theorem lipschitz_uniform_upper_minus_lower(f: Real -> Real, a: Real, b: Real, n: Nat) {
    a <= b and
    (forall(u: Real, v: Real) { (f(u) - f(v)).abs <= (u - v).abs }) and
    (forall(t: Real) { interval_contains(a, b, t) implies -Real.1 <= f(t) }) and
    (forall(t: Real) { interval_contains(a, b, t) implies f(t) <= Real.1 }) implies
    upper_sum(f, uniform_partition(a, b, n), n.suc) -
        lower_sum(f, uniform_partition(a, b, n), n.suc) <= ((b - a) * (b - a)) / from_nat[Real](n.suc)
} by {
    if a <= b and
       (forall(u: Real, v: Real) { (f(u) - f(v)).abs <= (u - v).abs }) and
       (forall(t: Real) { interval_contains(a, b, t) implies -Real.1 <= f(t) }) and
       (forall(t: Real) { interval_contains(a, b, t) implies f(t) <= Real.1 }) {
        uniform_partition_is_partition(a, b, n)
        is_partition(uniform_partition(a, b, n), a, b, n.suc)
        uniform_mesh(a, b, n) = (b - a) / from_nat[Real](n.suc)
        forall(i: Nat) {
            if i < n.suc {
                uniform_partition_width(a, b, n, i)
                uniform_partition(a, b, n, i + 1) - uniform_partition(a, b, n, i) =
                    (b - a) / from_nat[Real](n.suc)
                uniform_partition(a, b, n, i + 1) - uniform_partition(a, b, n, i) =
                    uniform_mesh(a, b, n)
                lt_imp_lte_suc(i, n.suc)
                i + 1 <= n.suc
                i <= i + 1
                lt_imp_lte(i, n.suc)
                i <= n.suc
                partition_mono(uniform_partition(a, b, n), a, b, n.suc, i, i + 1)
                uniform_partition(a, b, n, i) <= uniform_partition(a, b, n, i + 1)
                interval_set_contains_left(uniform_partition(a, b, n, i), uniform_partition(a, b, n, i + 1))
                interval_set(uniform_partition(a, b, n, i), uniform_partition(a, b, n, i + 1)).contains(
                    uniform_partition(a, b, n, i))
                exists(z: Real) {
                    interval_set(uniform_partition(a, b, n, i), uniform_partition(a, b, n, i + 1)).contains(z)
                }
                is_nonempty(interval_set(uniform_partition(a, b, n, i), uniform_partition(a, b, n, i + 1)))
                forall(t: Real) {
                    if interval_contains(uniform_partition(a, b, n, i), uniform_partition(a, b, n, i + 1), t) {
                        interval_contains_left(uniform_partition(a, b, n, i), uniform_partition(a, b, n, i + 1), t)
                        uniform_partition(a, b, n, i) <= t
                        interval_contains_right(uniform_partition(a, b, n, i), uniform_partition(a, b, n, i + 1), t)
                        t <= uniform_partition(a, b, n, i + 1)
                        partition_point_in_interval(uniform_partition(a, b, n), a, b, n.suc, i)
                        interval_contains(a, b, uniform_partition(a, b, n, i))
                        partition_point_in_interval(uniform_partition(a, b, n), a, b, n.suc, i + 1)
                        interval_contains(a, b, uniform_partition(a, b, n, i + 1))
                        interval_contains_mono(a, b, uniform_partition(a, b, n, i),
                            uniform_partition(a, b, n, i + 1), t)
                        interval_contains(a, b, t)
                        forall(t0: Real) {
                            interval_contains(a, b, t0) implies f(t0) <= Real.1
                        }
                        interval_contains(a, b, t) implies f(t) <= Real.1
                        f(t) <= Real.1
                    }
                }
                image_upper_bound(f, uniform_partition(a, b, n, i), uniform_partition(a, b, n, i + 1), Real.1)
                is_set_upper_bound(interval_image(f, uniform_partition(a, b, n, i),
                    uniform_partition(a, b, n, i + 1)), Real.1)
                exists(b0: Real) {
                    is_set_upper_bound(interval_image(f, uniform_partition(a, b, n, i),
                        uniform_partition(a, b, n, i + 1)), b0)
                }
                has_upper_bound(interval_image(f, uniform_partition(a, b, n, i),
                    uniform_partition(a, b, n, i + 1)))
                forall(t: Real) {
                    if interval_contains(uniform_partition(a, b, n, i), uniform_partition(a, b, n, i + 1), t) {
                        interval_contains_left(uniform_partition(a, b, n, i), uniform_partition(a, b, n, i + 1), t)
                        uniform_partition(a, b, n, i) <= t
                        interval_contains_right(uniform_partition(a, b, n, i), uniform_partition(a, b, n, i + 1), t)
                        t <= uniform_partition(a, b, n, i + 1)
                        partition_point_in_interval(uniform_partition(a, b, n), a, b, n.suc, i)
                        interval_contains(a, b, uniform_partition(a, b, n, i))
                        partition_point_in_interval(uniform_partition(a, b, n), a, b, n.suc, i + 1)
                        interval_contains(a, b, uniform_partition(a, b, n, i + 1))
                        interval_contains_mono(a, b, uniform_partition(a, b, n, i),
                            uniform_partition(a, b, n, i + 1), t)
                        interval_contains(a, b, t)
                        forall(t0: Real) {
                            interval_contains(a, b, t0) implies -Real.1 <= f(t0)
                        }
                        interval_contains(a, b, t) implies -Real.1 <= f(t)
                        -Real.1 <= f(t)
                    }
                }
                image_lower_bound(f, uniform_partition(a, b, n, i), uniform_partition(a, b, n, i + 1), -Real.1)
                is_set_lower_bound(interval_image(f, uniform_partition(a, b, n, i),
                    uniform_partition(a, b, n, i + 1)), -Real.1)
                exists(b0: Real) {
                    is_set_lower_bound(interval_image(f, uniform_partition(a, b, n, i),
                        uniform_partition(a, b, n, i + 1)), b0)
                }
                has_lower_bound(interval_image(f, uniform_partition(a, b, n, i),
                    uniform_partition(a, b, n, i + 1)))
                forall(u: Real, v: Real) {
                    if interval_contains(uniform_partition(a, b, n, i), uniform_partition(a, b, n, i + 1), u) and
                       interval_contains(uniform_partition(a, b, n, i), uniform_partition(a, b, n, i + 1), v) {
                        interval_contains_left(uniform_partition(a, b, n, i), uniform_partition(a, b, n, i + 1), u)
                        uniform_partition(a, b, n, i) <= u
                        interval_contains_right(uniform_partition(a, b, n, i), uniform_partition(a, b, n, i + 1), u)
                        u <= uniform_partition(a, b, n, i + 1)
                        interval_contains_left(uniform_partition(a, b, n, i), uniform_partition(a, b, n, i + 1), v)
                        uniform_partition(a, b, n, i) <= v
                        interval_contains_right(uniform_partition(a, b, n, i), uniform_partition(a, b, n, i + 1), v)
                        v <= uniform_partition(a, b, n, i + 1)
                        interval_abs_diff_le_width(uniform_partition(a, b, n, i),
                            uniform_partition(a, b, n, i + 1), u, v)
                        (u - v).abs <= uniform_partition(a, b, n, i + 1) - uniform_partition(a, b, n, i)
                        (u - v).abs <= uniform_mesh(a, b, n)
                        forall(u0: Real, v0: Real) {
                            (f(u0) - f(v0)).abs <= (u0 - v0).abs
                        }
                        (f(u) - f(v)).abs <= (u - v).abs
                        lte_trans[Real]((f(u) - f(v)).abs, (u - v).abs, uniform_mesh(a, b, n))
                        (f(u) - f(v)).abs <= uniform_mesh(a, b, n)
                        (f(u) - f(v)).abs <= uniform_partition(a, b, n, i + 1) - uniform_partition(a, b, n, i)
                    }
                }
                interval_sup_sub_inf_le_lipschitz(f, uniform_partition(a, b, n, i),
                    uniform_partition(a, b, n, i + 1))
                interval_sup(f, uniform_partition(a, b, n, i), uniform_partition(a, b, n, i + 1)) -
                    interval_inf(f, uniform_partition(a, b, n, i), uniform_partition(a, b, n, i + 1)) <= uniform_partition(a, b, n, i + 1) - uniform_partition(a, b, n, i)
                interval_sup(f, uniform_partition(a, b, n, i), uniform_partition(a, b, n, i + 1)) -
                    interval_inf(f, uniform_partition(a, b, n, i), uniform_partition(a, b, n, i + 1)) <= uniform_mesh(a, b, n)
                partition_step_upper(f, uniform_partition(a, b, n), i) =
                    interval_sup(f, uniform_partition(a, b, n, i), uniform_partition(a, b, n, i + 1)) *
                        diff_step(uniform_partition(a, b, n), i)
                diff_step(uniform_partition(a, b, n), i) =
                    uniform_partition(a, b, n, i + 1) - uniform_partition(a, b, n, i)
                diff_step(uniform_partition(a, b, n), i) = uniform_mesh(a, b, n)
                partition_step_upper(f, uniform_partition(a, b, n), i) =
                    interval_sup(f, uniform_partition(a, b, n, i), uniform_partition(a, b, n, i + 1)) *
                        uniform_mesh(a, b, n)
                partition_step_lower(f, uniform_partition(a, b, n), i) =
                    interval_inf(f, uniform_partition(a, b, n, i), uniform_partition(a, b, n, i + 1)) *
                        diff_step(uniform_partition(a, b, n), i)
                partition_step_lower(f, uniform_partition(a, b, n), i) =
                    interval_inf(f, uniform_partition(a, b, n, i), uniform_partition(a, b, n, i + 1)) *
                        uniform_mesh(a, b, n)
                partition_step_upper(f, uniform_partition(a, b, n), i) -
                    partition_step_lower(f, uniform_partition(a, b, n), i) =
                    (interval_sup(f, uniform_partition(a, b, n, i), uniform_partition(a, b, n, i + 1)) -
                        interval_inf(f, uniform_partition(a, b, n, i), uniform_partition(a, b, n, i + 1))) *
                        uniform_mesh(a, b, n)
                from_nat_suc_pos_real(n)
                from_nat[Real](n.suc) > Real.0
                from_nat[Real](n.suc) != Real.0
                sub_nonneg(a, b)
                Real.0 <= b - a
                div_nonneg_pos_denom(b - a, from_nat[Real](n.suc))
                Real.0 <= uniform_mesh(a, b, n)
                mul_le_mul_of_nonneg_right(
                    interval_sup(f, uniform_partition(a, b, n, i), uniform_partition(a, b, n, i + 1)) -
                        interval_inf(f, uniform_partition(a, b, n, i), uniform_partition(a, b, n, i + 1)),
                    uniform_mesh(a, b, n), uniform_mesh(a, b, n))
                (interval_sup(f, uniform_partition(a, b, n, i), uniform_partition(a, b, n, i + 1)) -
                    interval_inf(f, uniform_partition(a, b, n, i), uniform_partition(a, b, n, i + 1))) *
                    uniform_mesh(a, b, n) <= uniform_mesh(a, b, n) * uniform_mesh(a, b, n)
                partition_step_upper(f, uniform_partition(a, b, n), i) -
                    partition_step_lower(f, uniform_partition(a, b, n), i) <= uniform_mesh(a, b, n) * uniform_mesh(a, b, n)
                sub_seq(partition_step_upper(f, uniform_partition(a, b, n)),
                    partition_step_lower(f, uniform_partition(a, b, n)), i) =
                    partition_step_upper(f, uniform_partition(a, b, n), i) -
                    partition_step_lower(f, uniform_partition(a, b, n), i)
                sub_seq(partition_step_upper(f, uniform_partition(a, b, n)),
                    partition_step_lower(f, uniform_partition(a, b, n)), i) <= uniform_mesh(a, b, n) * uniform_mesh(a, b, n)
            }
        }
        partial_lte(sub_seq(partition_step_upper(f, uniform_partition(a, b, n)),
                partition_step_lower(f, uniform_partition(a, b, n))),
            const_seq(uniform_mesh(a, b, n) * uniform_mesh(a, b, n)), n.suc)
        partial(sub_seq(partition_step_upper(f, uniform_partition(a, b, n)),
                partition_step_lower(f, uniform_partition(a, b, n))), n.suc) <= partial(const_seq(uniform_mesh(a, b, n) * uniform_mesh(a, b, n)), n.suc)
        forall(j: Nat) {
            if j < n.suc {
                const_seq(uniform_mesh(a, b, n) * uniform_mesh(a, b, n), j) =
                    uniform_mesh(a, b, n) * uniform_mesh(a, b, n)
            }
        }
        partial_const(const_seq(uniform_mesh(a, b, n) * uniform_mesh(a, b, n)), n.suc,
            uniform_mesh(a, b, n) * uniform_mesh(a, b, n))
        partial(const_seq(uniform_mesh(a, b, n) * uniform_mesh(a, b, n)), n.suc) =
            from_nat[Real](n.suc) * (uniform_mesh(a, b, n) * uniform_mesh(a, b, n))
        partial(sub_seq(partition_step_upper(f, uniform_partition(a, b, n)),
                partition_step_lower(f, uniform_partition(a, b, n))), n.suc) <= from_nat[Real](n.suc) * (uniform_mesh(a, b, n) * uniform_mesh(a, b, n))
        partial_sub_seq(partition_step_upper(f, uniform_partition(a, b, n)),
            partition_step_lower(f, uniform_partition(a, b, n)), n.suc)
        partial(sub_seq(partition_step_upper(f, uniform_partition(a, b, n)),
                partition_step_lower(f, uniform_partition(a, b, n))), n.suc) =
            partial(partition_step_upper(f, uniform_partition(a, b, n)), n.suc) -
            partial(partition_step_lower(f, uniform_partition(a, b, n)), n.suc)
        partial(partition_step_upper(f, uniform_partition(a, b, n)), n.suc) -
            partial(partition_step_lower(f, uniform_partition(a, b, n)), n.suc) <= from_nat[Real](n.suc) * (uniform_mesh(a, b, n) * uniform_mesh(a, b, n))
        from_nat_suc_pos_real(n)
        from_nat[Real](n.suc) > Real.0
        from_nat[Real](n.suc) != Real.0
        div_mul_cancel_denominator(b - a, from_nat[Real](n.suc))
        ((b - a) / from_nat[Real](n.suc)) * from_nat[Real](n.suc) = b - a
        uniform_mesh(a, b, n) * from_nat[Real](n.suc) = b - a
        from_nat[Real](n.suc) * (uniform_mesh(a, b, n) * uniform_mesh(a, b, n)) =
            (from_nat[Real](n.suc) * uniform_mesh(a, b, n)) * uniform_mesh(a, b, n)
        from_nat[Real](n.suc) * uniform_mesh(a, b, n) = b - a
        from_nat[Real](n.suc) * (uniform_mesh(a, b, n) * uniform_mesh(a, b, n)) =
            (b - a) * uniform_mesh(a, b, n)
        mul_frac_right(b - a, from_nat[Real](n.suc), b - a)
        ((b - a) / from_nat[Real](n.suc)) * (b - a) = ((b - a) * (b - a)) / from_nat[Real](n.suc)
        (b - a) * uniform_mesh(a, b, n) = ((b - a) * (b - a)) / from_nat[Real](n.suc)
        from_nat[Real](n.suc) * (uniform_mesh(a, b, n) * uniform_mesh(a, b, n)) =
            ((b - a) * (b - a)) / from_nat[Real](n.suc)
        partial(partition_step_upper(f, uniform_partition(a, b, n)), n.suc) -
            partial(partition_step_lower(f, uniform_partition(a, b, n)), n.suc) <= ((b - a) * (b - a)) / from_nat[Real](n.suc)
        upper_sum(f, uniform_partition(a, b, n), n.suc) =
            partial(partition_step_upper(f, uniform_partition(a, b, n)), n.suc)
        lower_sum(f, uniform_partition(a, b, n), n.suc) =
            partial(partition_step_lower(f, uniform_partition(a, b, n)), n.suc)
        upper_sum(f, uniform_partition(a, b, n), n.suc) -
            lower_sum(f, uniform_partition(a, b, n), n.suc) <= ((b - a) * (b - a)) / from_nat[Real](n.suc)
    }
}

// ---------------------------------------------------------------------------
// Integrability of functions with known antiderivative
// ---------------------------------------------------------------------------
//
// Following integral_exp.ac: the lower sums of f over [a, b] have a supremum
// l and the upper sums an infimum u, both sandwiched by g(b) - g(a) when
// g' = f; the uniform partition forces u - l below ((b - a) * (b - a)) / (n + 1)
// for every n, and the Archimedean property then forces u = l, which is
// integrability.

/// The lower sums of f over [a, b] are bounded above by g(b) - g(a) when g'
/// is f and f is bounded below on [a, b].
theorem fn_lower_sum_set_bounded_above(f: Real -> Real, g: Real -> Real, a: Real, b: Real, lb: Real) {
    a <= b and continuous(g) and is_derivative_fn(g, f) and
    (forall(t: Real) { interval_contains(a, b, t) implies lb <= f(t) }) implies
    is_set_upper_bound(lower_sum_set(f, a, b), g(b) - g(a))
} by {
    if a <= b and continuous(g) and is_derivative_fn(g, f) and
       (forall(t: Real) { interval_contains(a, b, t) implies lb <= f(t) }) {
        forall(s: Real) {
            if lower_sum_set(f, a, b).contains(s) {
                lower_sum_set(f, a, b).contains(s) = lower_sum_contains(f, a, b, s)
                lower_sum_contains(f, a, b, s)
                let (p: Nat -> Real, n: Nat) satisfy {
                    is_partition(p, a, b, n) and s = lower_sum(f, p, n)
                }
                is_partition(p, a, b, n)
                lower_sum_le_g_diff(f, g, a, b, lb, p, n)
                lower_sum(f, p, n) <= g(b) - g(a)
                s = lower_sum(f, p, n)
                s <= g(b) - g(a)
            }
        }
        is_set_upper_bound(lower_sum_set(f, a, b), g(b) - g(a))
    }
}

/// The upper sums of f over [a, b] are bounded below by g(b) - g(a) when g'
/// is f and f is bounded above on [a, b].
theorem fn_upper_sum_set_bounded_below(f: Real -> Real, g: Real -> Real, a: Real, b: Real, ub: Real) {
    a <= b and continuous(g) and is_derivative_fn(g, f) and
    (forall(t: Real) { interval_contains(a, b, t) implies f(t) <= ub }) implies
    is_set_lower_bound(upper_sum_set(f, a, b), g(b) - g(a))
} by {
    if a <= b and continuous(g) and is_derivative_fn(g, f) and
       (forall(t: Real) { interval_contains(a, b, t) implies f(t) <= ub }) {
        forall(s: Real) {
            if upper_sum_set(f, a, b).contains(s) {
                upper_sum_set(f, a, b).contains(s) = upper_sum_contains(f, a, b, s)
                upper_sum_contains(f, a, b, s)
                let (p: Nat -> Real, n: Nat) satisfy {
                    is_partition(p, a, b, n) and s = upper_sum(f, p, n)
                }
                is_partition(p, a, b, n)
                g_diff_le_upper_sum(f, g, a, b, ub, p, n)
                g(b) - g(a) <= upper_sum(f, p, n)
                s = upper_sum(f, p, n)
                g(b) - g(a) <= s
            }
        }
        is_set_lower_bound(upper_sum_set(f, a, b), g(b) - g(a))
    }
}

/// The set of lower sums of f over [a, b] has a supremum.
theorem fn_lower_sum_set_sup_exists(f: Real -> Real, g: Real -> Real, a: Real, b: Real, lb: Real) {
    a <= b and continuous(g) and is_derivative_fn(g, f) and
    (forall(t: Real) { interval_contains(a, b, t) implies lb <= f(t) }) implies exists(l: Real) {
        is_set_supremum(lower_sum_set(f, a, b), l)
    }
} by {
    if a <= b and continuous(g) and is_derivative_fn(g, f) and
       (forall(t: Real) { interval_contains(a, b, t) implies lb <= f(t) }) {
        trivial_partition_is_partition(a, b)
        is_partition(trivial_partition(a, b), a, b, Nat.1)
        is_partition(trivial_partition(a, b), a, b, Nat.1) and
            lower_sum(f, trivial_partition(a, b), Nat.1) = lower_sum(f, trivial_partition(a, b), Nat.1)
        exists(p: Nat -> Real, n: Nat) {
            is_partition(p, a, b, n) and lower_sum(f, trivial_partition(a, b), Nat.1) = lower_sum(f, p, n)
        }
        lower_sum_contains(f, a, b, lower_sum(f, trivial_partition(a, b), Nat.1))
        lower_sum_set(f, a, b).contains(lower_sum(f, trivial_partition(a, b), Nat.1)) =
            lower_sum_contains(f, a, b, lower_sum(f, trivial_partition(a, b), Nat.1))
        lower_sum_set(f, a, b).contains(lower_sum(f, trivial_partition(a, b), Nat.1))
        exists(x: Real) {
            lower_sum_set(f, a, b).contains(x)
        }
        is_nonempty(lower_sum_set(f, a, b))
        fn_lower_sum_set_bounded_above(f, g, a, b, lb)
        is_set_upper_bound(lower_sum_set(f, a, b), g(b) - g(a))
        exists(b0: Real) {
            is_set_upper_bound(lower_sum_set(f, a, b), b0)
        }
        has_upper_bound(lower_sum_set(f, a, b))
        is_nonempty(lower_sum_set(f, a, b)) and has_upper_bound(lower_sum_set(f, a, b))
        completeness(lower_sum_set(f, a, b))
        let l: Real satisfy {
            is_set_supremum(lower_sum_set(f, a, b), l)
        }
        exists(l2: Real) {
            is_set_supremum(lower_sum_set(f, a, b), l2)
        }
    }
}

/// The set of upper sums of f over [a, b] has an infimum.
theorem fn_upper_sum_set_inf_exists(f: Real -> Real, g: Real -> Real, a: Real, b: Real, ub: Real) {
    a <= b and continuous(g) and is_derivative_fn(g, f) and
    (forall(t: Real) { interval_contains(a, b, t) implies f(t) <= ub }) implies exists(u: Real) {
        is_set_infimum(upper_sum_set(f, a, b), u)
    }
} by {
    if a <= b and continuous(g) and is_derivative_fn(g, f) and
       (forall(t: Real) { interval_contains(a, b, t) implies f(t) <= ub }) {
        trivial_partition_is_partition(a, b)
        is_partition(trivial_partition(a, b), a, b, Nat.1)
        is_partition(trivial_partition(a, b), a, b, Nat.1) and
            upper_sum(f, trivial_partition(a, b), Nat.1) = upper_sum(f, trivial_partition(a, b), Nat.1)
        exists(p: Nat -> Real, n: Nat) {
            is_partition(p, a, b, n) and upper_sum(f, trivial_partition(a, b), Nat.1) = upper_sum(f, p, n)
        }
        upper_sum_contains(f, a, b, upper_sum(f, trivial_partition(a, b), Nat.1))
        upper_sum_set(f, a, b).contains(upper_sum(f, trivial_partition(a, b), Nat.1)) =
            upper_sum_contains(f, a, b, upper_sum(f, trivial_partition(a, b), Nat.1))
        upper_sum_set(f, a, b).contains(upper_sum(f, trivial_partition(a, b), Nat.1))
        exists(x: Real) {
            upper_sum_set(f, a, b).contains(x)
        }
        is_nonempty(upper_sum_set(f, a, b))
        negate_set_nonempty(upper_sum_set(f, a, b))
        is_nonempty(negate_set(upper_sum_set(f, a, b)))
        fn_upper_sum_set_bounded_below(f, g, a, b, ub)
        is_set_lower_bound(upper_sum_set(f, a, b), g(b) - g(a))
        neg_upper_bound_of_lower(upper_sum_set(f, a, b), g(b) - g(a))
        is_set_upper_bound(negate_set(upper_sum_set(f, a, b)), -(g(b) - g(a)))
        exists(b0: Real) {
            is_set_upper_bound(negate_set(upper_sum_set(f, a, b)), b0)
        }
        has_upper_bound(negate_set(upper_sum_set(f, a, b)))
        is_nonempty(negate_set(upper_sum_set(f, a, b))) and has_upper_bound(negate_set(upper_sum_set(f, a, b)))
        completeness(negate_set(upper_sum_set(f, a, b)))
        let sup: Real satisfy {
            is_set_supremum(negate_set(upper_sum_set(f, a, b)), sup)
        }
        inf_of_neg_sup(upper_sum_set(f, a, b), sup)
        is_set_infimum(upper_sum_set(f, a, b), -sup)
        exists(u2: Real) {
            is_set_infimum(upper_sum_set(f, a, b), u2)
        }
    }
}

/// A one-Lipschitz f bounded by one in absolute value on [a, b], with known
/// antiderivative g, is integrable on [a, b].
theorem fn_integrable(f: Real -> Real, g: Real -> Real, a: Real, b: Real) {
    a <= b and continuous(g) and is_derivative_fn(g, f) and
    (forall(u: Real, v: Real) { (f(u) - f(v)).abs <= (u - v).abs }) and
    (forall(t: Real) { interval_contains(a, b, t) implies -Real.1 <= f(t) }) and
    (forall(t: Real) { interval_contains(a, b, t) implies f(t) <= Real.1 }) implies
    is_integrable(f, a, b)
} by {
    if a <= b and continuous(g) and is_derivative_fn(g, f) and
       (forall(u: Real, v: Real) { (f(u) - f(v)).abs <= (u - v).abs }) and
       (forall(t: Real) { interval_contains(a, b, t) implies -Real.1 <= f(t) }) and
       (forall(t: Real) { interval_contains(a, b, t) implies f(t) <= Real.1 }) {
        fn_lower_sum_set_sup_exists(f, g, a, b, -Real.1)
        let l: Real satisfy {
            is_set_supremum(lower_sum_set(f, a, b), l)
        }
        fn_upper_sum_set_inf_exists(f, g, a, b, Real.1)
        let u: Real satisfy {
            is_set_infimum(upper_sum_set(f, a, b), u)
        }
        fn_lower_sum_set_bounded_above(f, g, a, b, -Real.1)
        is_set_upper_bound(lower_sum_set(f, a, b), g(b) - g(a))
        set_supremum_le_upper_bound(lower_sum_set(f, a, b), l, g(b) - g(a))
        l <= g(b) - g(a)
        fn_upper_sum_set_bounded_below(f, g, a, b, Real.1)
        is_set_lower_bound(upper_sum_set(f, a, b), g(b) - g(a))
        set_lower_bound_le_infimum(upper_sum_set(f, a, b), u, g(b) - g(a))
        g(b) - g(a) <= u
        lte_trans[Real](l, g(b) - g(a), u)
        l <= u
        sub_nonneg(l, u)
        Real.0 <= u - l
        forall(n: Nat) {
            lipschitz_uniform_upper_minus_lower(f, a, b, n)
            upper_sum(f, uniform_partition(a, b, n), n.suc) -
                lower_sum(f, uniform_partition(a, b, n), n.suc) <= ((b - a) * (b - a)) / from_nat[Real](n.suc)
            uniform_partition_is_partition(a, b, n)
            is_partition(uniform_partition(a, b, n), a, b, n.suc)
            is_partition(uniform_partition(a, b, n), a, b, n.suc) and
                lower_sum(f, uniform_partition(a, b, n), n.suc) =
                    lower_sum(f, uniform_partition(a, b, n), n.suc)
            exists(p0: Nat -> Real, n0: Nat) {
                is_partition(p0, a, b, n0) and
                    lower_sum(f, uniform_partition(a, b, n), n.suc) = lower_sum(f, p0, n0)
            }
            lower_sum_contains(f, a, b, lower_sum(f, uniform_partition(a, b, n), n.suc))
            lower_sum_set(f, a, b).contains(lower_sum(f, uniform_partition(a, b, n), n.suc)) =
                lower_sum_contains(f, a, b, lower_sum(f, uniform_partition(a, b, n), n.suc))
            lower_sum_set(f, a, b).contains(lower_sum(f, uniform_partition(a, b, n), n.suc))
            set_member_le_supremum(lower_sum_set(f, a, b), l,
                lower_sum(f, uniform_partition(a, b, n), n.suc))
            lower_sum(f, uniform_partition(a, b, n), n.suc) <= l
            is_partition(uniform_partition(a, b, n), a, b, n.suc) and
                upper_sum(f, uniform_partition(a, b, n), n.suc) =
                    upper_sum(f, uniform_partition(a, b, n), n.suc)
            exists(p1: Nat -> Real, n1: Nat) {
                is_partition(p1, a, b, n1) and
                    upper_sum(f, uniform_partition(a, b, n), n.suc) = upper_sum(f, p1, n1)
            }
            upper_sum_contains(f, a, b, upper_sum(f, uniform_partition(a, b, n), n.suc))
            upper_sum_set(f, a, b).contains(upper_sum(f, uniform_partition(a, b, n), n.suc)) =
                upper_sum_contains(f, a, b, upper_sum(f, uniform_partition(a, b, n), n.suc))
            upper_sum_set(f, a, b).contains(upper_sum(f, uniform_partition(a, b, n), n.suc))
            set_infimum_is_lower_bound(upper_sum_set(f, a, b), u)
            is_set_lower_bound(upper_sum_set(f, a, b), u)
            set_lower_bound_contains_le(upper_sum_set(f, a, b), u,
                upper_sum(f, uniform_partition(a, b, n), n.suc))
            u <= upper_sum(f, uniform_partition(a, b, n), n.suc)
            neg_lte_flip(lower_sum(f, uniform_partition(a, b, n), n.suc), l)
            -l <= -lower_sum(f, uniform_partition(a, b, n), n.suc)
            add_le_add(u, upper_sum(f, uniform_partition(a, b, n), n.suc),
                -l, -lower_sum(f, uniform_partition(a, b, n), n.suc))
            u + -l <= upper_sum(f, uniform_partition(a, b, n), n.suc) +
                -lower_sum(f, uniform_partition(a, b, n), n.suc)
            u - l = u + -l
            upper_sum(f, uniform_partition(a, b, n), n.suc) -
                lower_sum(f, uniform_partition(a, b, n), n.suc) =
                upper_sum(f, uniform_partition(a, b, n), n.suc) +
                -lower_sum(f, uniform_partition(a, b, n), n.suc)
            u - l <= upper_sum(f, uniform_partition(a, b, n), n.suc) -
                lower_sum(f, uniform_partition(a, b, n), n.suc)
            lte_trans[Real](u - l,
                upper_sum(f, uniform_partition(a, b, n), n.suc) -
                    lower_sum(f, uniform_partition(a, b, n), n.suc),
                ((b - a) * (b - a)) / from_nat[Real](n.suc))
            u - l <= ((b - a) * (b - a)) / from_nat[Real](n.suc)
        }
        nonneg_frac_all_n_imp_zero((b - a) * (b - a), u - l)
        u - l = Real.0
        sub_zero_imp_eq(u, l)
        u = l
        is_set_infimum(upper_sum_set(f, a, b), l)
        is_set_supremum(lower_sum_set(f, a, b), l) and is_set_infimum(upper_sum_set(f, a, b), l)
        exists(m: Real) {
            is_set_supremum(lower_sum_set(f, a, b), m) and is_set_infimum(upper_sum_set(f, a, b), m)
        }
        is_integrable(f, a, b)
    }
}

// ---------------------------------------------------------------------------
// The integrals of sine and cosine
// ---------------------------------------------------------------------------

/// Sine is integrable on every closed interval [a, b].
theorem sin_integrable(a: Real, b: Real) {
    a <= b implies is_integrable(Real.sin, a, b)
} by {
    if a <= b {
        neg_cos_continuous
        continuous(pointwise_neg(Real.cos))
        neg_cos_is_derivative_fn
        is_derivative_fn(pointwise_neg(Real.cos), Real.sin)
        forall(u: Real, v: Real) {
            sin_lipschitz(u, v)
            (u.sin - v.sin).abs <= (u - v).abs
        }
        sin_lower_bound_on(a, b)
        sin_upper_bound_on(a, b)
        fn_integrable(Real.sin, pointwise_neg(Real.cos), a, b)
        is_integrable(Real.sin, a, b)
    }
}

/// Cosine is integrable on every closed interval [a, b].
theorem cos_integrable(a: Real, b: Real) {
    a <= b implies is_integrable(Real.cos, a, b)
} by {
    if a <= b {
        sin_continuous
        continuous(Real.sin)
        sin_is_derivative_fn
        is_derivative_fn(Real.sin, Real.cos)
        forall(u: Real, v: Real) {
            cos_lipschitz(u, v)
            (u.cos - v.cos).abs <= (u - v).abs
        }
        cos_lower_bound_on(a, b)
        cos_upper_bound_on(a, b)
        fn_integrable(Real.cos, Real.sin, a, b)
        is_integrable(Real.cos, a, b)
    }
}

/// The integral of sine over [a, b] is a.cos - b.cos.
theorem integral_sin(a: Real, b: Real) {
    a <= b implies integral(Real.sin, a, b) = a.cos - b.cos
} by {
    if a <= b {
        sin_integrable(a, b)
        is_integrable(Real.sin, a, b)
        neg_cos_continuous
        continuous(pointwise_neg(Real.cos))
        neg_cos_is_derivative_fn
        is_derivative_fn(pointwise_neg(Real.cos), Real.sin)
        sin_lower_bound_on(a, b)
        sin_upper_bound_on(a, b)
        ftc2_general(Real.sin, pointwise_neg(Real.cos), a, b, -Real.1, Real.1)
        integral(Real.sin, a, b) = pointwise_neg(Real.cos, b) - pointwise_neg(Real.cos, a)
        pointwise_neg(Real.cos, b) = -b.cos
        pointwise_neg(Real.cos, a) = -a.cos
        integral(Real.sin, a, b) = -b.cos - (-a.cos)
        -b.cos - (-a.cos) = a.cos - b.cos
        integral(Real.sin, a, b) = a.cos - b.cos
    }
}

/// The integral of cosine over [a, b] is b.sin - a.sin.
theorem integral_cos(a: Real, b: Real) {
    a <= b implies integral(Real.cos, a, b) = b.sin - a.sin
} by {
    if a <= b {
        cos_integrable(a, b)
        is_integrable(Real.cos, a, b)
        sin_continuous
        continuous(Real.sin)
        sin_is_derivative_fn
        is_derivative_fn(Real.sin, Real.cos)
        cos_lower_bound_on(a, b)
        cos_upper_bound_on(a, b)
        ftc2_general(Real.cos, Real.sin, a, b, -Real.1, Real.1)
        integral(Real.cos, a, b) = b.sin - a.sin
    }
}

// ---------------------------------------------------------------------------
// Values
// ---------------------------------------------------------------------------

/// The integral of sine over [0, pi] is two.
theorem integral_sin_zero_pi {
    integral(Real.sin, Real.0, pi) = two
} by {
    pi_pos
    pi > Real.0
    lt_imp_lte(Real.0, pi)
    Real.0 <= pi
    integral_sin(Real.0, pi)
    integral(Real.sin, Real.0, pi) = (Real.0).cos - pi.cos
    cos_zero
    (Real.0).cos = Real.1
    cos_pi_neg_one
    pi.cos = -Real.1
    (Real.0).cos - pi.cos = Real.1 - (-Real.1)
    Real.1 - (-Real.1) = Real.1 + Real.1
    Real.1 + Real.1 = two
    Real.1 - (-Real.1) = two
    integral(Real.sin, Real.0, pi) = two
}

/// The integral of cosine over [0, pi/2] is one.
theorem integral_cos_zero_pi_over_two {
    integral(Real.cos, Real.0, pi_over_two) = Real.1
} by {
    pi_over_two_pos
    Real.0 < pi_over_two
    lt_imp_lte(Real.0, pi_over_two)
    Real.0 <= pi_over_two
    integral_cos(Real.0, pi_over_two)
    integral(Real.cos, Real.0, pi_over_two) = pi_over_two.sin - (Real.0).sin
    sin_pi_over_two_one
    pi_over_two.sin = Real.1
    sin_zero
    (Real.0).sin = Real.0
    pi_over_two.sin - (Real.0).sin = Real.1 - Real.0
    Real.1 - Real.0 = Real.1
    integral(Real.cos, Real.0, pi_over_two) = Real.1
}

// ---------------------------------------------------------------------------
// Integrability of functions with an m-Lipschitz bound
// ---------------------------------------------------------------------------
//
// The one-Lipschitz criterion fn_integrable above is generalized here to
// functions that are m-Lipschitz on [a, b] for a fixed m >= 0 and bounded on
// [a, b].  The argument is identical to the one-Lipschitz case: the
// mean-value-theorem sandwich pins the Darboux sums between the increments of
// an antiderivative, and the m-Lipschitz bound forces the upper and lower
// sums over the uniform partitions to differ by at most
// m * (b - a)^2 / (n + 1), which goes to zero by the Archimedean property.

/// If f is m * (y - x)-Lipschitz on [x, y] and bounded there, its supremum
/// and infimum over [x, y] differ by at most m * (y - x).
theorem interval_sup_sub_inf_le_lipschitz_m(f: Real -> Real, m: Real, x: Real, y: Real) {
    x <= y and is_nonempty(interval_set(x, y)) and has_upper_bound(interval_image(f, x, y)) and
    has_lower_bound(interval_image(f, x, y)) and
    (forall(u: Real, v: Real) {
        interval_contains(x, y, u) and interval_contains(x, y, v) implies (f(u) - f(v)).abs <= m * (y - x)
    }) implies
    interval_sup(f, x, y) - interval_inf(f, x, y) <= m * (y - x)
} by {
    if x <= y and is_nonempty(interval_set(x, y)) and has_upper_bound(interval_image(f, x, y)) and
       has_lower_bound(interval_image(f, x, y)) and
       (forall(u: Real, v: Real) {
           interval_contains(x, y, u) and interval_contains(x, y, v) implies (f(u) - f(v)).abs <= m * (y - x)
       }) {
        interval_sup_spec(f, x, y)
        is_set_supremum(interval_image(f, x, y), interval_sup(f, x, y))
        forall(v: Real) {
            if interval_image(f, x, y).contains(v) {
                interval_image(f, x, y).contains(v) = function_image_contains(f, interval_set(x, y), v)
                function_image_contains(f, interval_set(x, y), v)
                let t: Real satisfy {
                    interval_set(x, y).contains(t) and v = f(t)
                }
                interval_set(x, y).contains(t) = interval_contains(x, y, t)
                interval_contains(x, y, t)
                forall(w: Real) {
                    if interval_image(f, x, y).contains(w) {
                        interval_image(f, x, y).contains(w) = function_image_contains(f, interval_set(x, y), w)
                        function_image_contains(f, interval_set(x, y), w)
                        let s: Real satisfy {
                            interval_set(x, y).contains(s) and w = f(s)
                        }
                        interval_set(x, y).contains(s) = interval_contains(x, y, s)
                        interval_contains(x, y, s)
                        forall(u: Real, v0: Real) {
                            interval_contains(x, y, u) and interval_contains(x, y, v0) implies (f(u) - f(v0)).abs <= m * (y - x)
                        }
                        interval_contains(x, y, s) and interval_contains(x, y, t) implies (f(s) - f(t)).abs <= m * (y - x)
                        interval_contains(x, y, s) and interval_contains(x, y, t)
                        (f(s) - f(t)).abs <= m * (y - x)
                        lte_abs(f(s) - f(t))
                        f(s) - f(t) <= (f(s) - f(t)).abs
                        lte_trans[Real](f(s) - f(t), (f(s) - f(t)).abs, m * (y - x))
                        f(s) - f(t) <= m * (y - x)
                        add_le_add_right[Real](f(s) - f(t), m * (y - x), f(t))
                        f(s) - f(t) + f(t) <= m * (y - x) + f(t)
                        f(s) - f(t) + f(t) = f(s)
                        f(s) <= m * (y - x) + f(t)
                        m * (y - x) + f(t) = f(t) + (m * (y - x))
                        f(s) <= f(t) + (m * (y - x))
                        w = f(s)
                        v = f(t)
                        w <= v + (m * (y - x))
                    }
                }
                is_set_upper_bound(interval_image(f, x, y), v + (m * (y - x)))
                sup_le_of_upper_bound(interval_image(f, x, y), interval_sup(f, x, y), v + (m * (y - x)))
                interval_sup(f, x, y) <= v + (m * (y - x))
                add_le_add_right[Real](interval_sup(f, x, y), v + (m * (y - x)), -(m * (y - x)))
                interval_sup(f, x, y) + -(m * (y - x)) <= v + (m * (y - x)) + -(m * (y - x))
                interval_sup(f, x, y) - (m * (y - x)) = interval_sup(f, x, y) + -(m * (y - x))
                v + (m * (y - x)) + -(m * (y - x)) = v + (m * (y - x)) - (m * (y - x))
                interval_sup(f, x, y) - (m * (y - x)) <= v + (m * (y - x)) - (m * (y - x))
                v + (m * (y - x)) - (m * (y - x)) = v
                interval_sup(f, x, y) - (m * (y - x)) <= v
            }
        }
        is_set_lower_bound(interval_image(f, x, y), interval_sup(f, x, y) - (m * (y - x)))
        interval_inf_spec(f, x, y)
        is_set_infimum(interval_image(f, x, y), interval_inf(f, x, y))
        set_lower_bound_le_infimum(interval_image(f, x, y), interval_inf(f, x, y),
            interval_sup(f, x, y) - (m * (y - x)))
        interval_sup(f, x, y) - (m * (y - x)) <= interval_inf(f, x, y)
        add_le_add_right[Real](interval_sup(f, x, y) - (m * (y - x)), interval_inf(f, x, y), m * (y - x))
        interval_sup(f, x, y) - (m * (y - x)) + (m * (y - x)) <= interval_inf(f, x, y) + (m * (y - x))
        interval_sup(f, x, y) - (m * (y - x)) + (m * (y - x)) = interval_sup(f, x, y)
        interval_sup(f, x, y) <= interval_inf(f, x, y) + (m * (y - x))
        add_le_add_right[Real](interval_sup(f, x, y), interval_inf(f, x, y) + (m * (y - x)),
            -interval_inf(f, x, y))
        interval_sup(f, x, y) + -interval_inf(f, x, y) <= interval_inf(f, x, y) + (m * (y - x)) + -interval_inf(f, x, y)
        interval_sup(f, x, y) - interval_inf(f, x, y) = interval_sup(f, x, y) + -interval_inf(f, x, y)
        interval_inf(f, x, y) + (m * (y - x)) + -interval_inf(f, x, y) =
            interval_inf(f, x, y) + (m * (y - x)) - interval_inf(f, x, y)
        add_comm(interval_inf(f, x, y), m * (y - x))
        interval_inf(f, x, y) + (m * (y - x)) = (m * (y - x)) + interval_inf(f, x, y)
        interval_inf(f, x, y) + (m * (y - x)) - interval_inf(f, x, y) =
            (m * (y - x)) + interval_inf(f, x, y) - interval_inf(f, x, y)
        sub_cancels(m * (y - x), interval_inf(f, x, y))
        (m * (y - x)) + interval_inf(f, x, y) - interval_inf(f, x, y) = m * (y - x)
        interval_inf(f, x, y) + (m * (y - x)) - interval_inf(f, x, y) = m * (y - x)
        interval_sup(f, x, y) - interval_inf(f, x, y) <= m * (y - x)
    }
}

/// If f is m-Lipschitz on [a, b] and bounded there, the difference of its
/// upper and lower Darboux sums over the uniform partition of [a, b] into
/// n + 1 equal parts is at most m * (b - a)^2 / (n + 1).
theorem lipschitz_uniform_upper_minus_lower_m(f: Real -> Real, m: Real, a: Real, b: Real, n: Nat, lb: Real, ub: Real) {
    a <= b and Real.0 <= m and
    (forall(u: Real, v: Real) {
        interval_contains(a, b, u) and interval_contains(a, b, v) implies (f(u) - f(v)).abs <= m * (u - v).abs
    }) and
    (forall(t: Real) { interval_contains(a, b, t) implies lb <= f(t) }) and
    (forall(t: Real) { interval_contains(a, b, t) implies f(t) <= ub }) implies
    upper_sum(f, uniform_partition(a, b, n), n.suc) -
        lower_sum(f, uniform_partition(a, b, n), n.suc) <= m * (((b - a) * (b - a)) / from_nat[Real](n.suc))
} by {
    if a <= b and Real.0 <= m and
       (forall(u: Real, v: Real) {
           interval_contains(a, b, u) and interval_contains(a, b, v) implies (f(u) - f(v)).abs <= m * (u - v).abs
       }) and
       (forall(t: Real) { interval_contains(a, b, t) implies lb <= f(t) }) and
       (forall(t: Real) { interval_contains(a, b, t) implies f(t) <= ub }) {
        uniform_partition_is_partition(a, b, n)
        is_partition(uniform_partition(a, b, n), a, b, n.suc)
        uniform_mesh(a, b, n) = (b - a) / from_nat[Real](n.suc)
        forall(i: Nat) {
            if i < n.suc {
                uniform_partition_width(a, b, n, i)
                uniform_partition(a, b, n, i + 1) - uniform_partition(a, b, n, i) =
                    (b - a) / from_nat[Real](n.suc)
                uniform_partition(a, b, n, i + 1) - uniform_partition(a, b, n, i) =
                    uniform_mesh(a, b, n)
                lt_imp_lte_suc(i, n.suc)
                i + 1 <= n.suc
                i <= i + 1
                lt_imp_lte(i, n.suc)
                i <= n.suc
                partition_mono(uniform_partition(a, b, n), a, b, n.suc, i, i + 1)
                uniform_partition(a, b, n, i) <= uniform_partition(a, b, n, i + 1)
                interval_set_contains_left(uniform_partition(a, b, n, i), uniform_partition(a, b, n, i + 1))
                interval_set(uniform_partition(a, b, n, i), uniform_partition(a, b, n, i + 1)).contains(
                    uniform_partition(a, b, n, i))
                exists(z: Real) {
                    interval_set(uniform_partition(a, b, n, i), uniform_partition(a, b, n, i + 1)).contains(z)
                }
                is_nonempty(interval_set(uniform_partition(a, b, n, i), uniform_partition(a, b, n, i + 1)))
                forall(t: Real) {
                    if interval_contains(uniform_partition(a, b, n, i), uniform_partition(a, b, n, i + 1), t) {
                        interval_contains_left(uniform_partition(a, b, n, i), uniform_partition(a, b, n, i + 1), t)
                        uniform_partition(a, b, n, i) <= t
                        interval_contains_right(uniform_partition(a, b, n, i), uniform_partition(a, b, n, i + 1), t)
                        t <= uniform_partition(a, b, n, i + 1)
                        partition_point_in_interval(uniform_partition(a, b, n), a, b, n.suc, i)
                        interval_contains(a, b, uniform_partition(a, b, n, i))
                        partition_point_in_interval(uniform_partition(a, b, n), a, b, n.suc, i + 1)
                        interval_contains(a, b, uniform_partition(a, b, n, i + 1))
                        interval_contains_mono(a, b, uniform_partition(a, b, n, i),
                            uniform_partition(a, b, n, i + 1), t)
                        interval_contains(a, b, t)
                        forall(t0: Real) {
                            interval_contains(a, b, t0) implies f(t0) <= ub
                        }
                        interval_contains(a, b, t) implies f(t) <= ub
                        f(t) <= ub
                    }
                }
                image_upper_bound(f, uniform_partition(a, b, n, i), uniform_partition(a, b, n, i + 1), ub)
                is_set_upper_bound(interval_image(f, uniform_partition(a, b, n, i),
                    uniform_partition(a, b, n, i + 1)), ub)
                exists(b0: Real) {
                    is_set_upper_bound(interval_image(f, uniform_partition(a, b, n, i),
                        uniform_partition(a, b, n, i + 1)), b0)
                }
                has_upper_bound(interval_image(f, uniform_partition(a, b, n, i),
                    uniform_partition(a, b, n, i + 1)))
                forall(t: Real) {
                    if interval_contains(uniform_partition(a, b, n, i), uniform_partition(a, b, n, i + 1), t) {
                        interval_contains_left(uniform_partition(a, b, n, i), uniform_partition(a, b, n, i + 1), t)
                        uniform_partition(a, b, n, i) <= t
                        interval_contains_right(uniform_partition(a, b, n, i), uniform_partition(a, b, n, i + 1), t)
                        t <= uniform_partition(a, b, n, i + 1)
                        partition_point_in_interval(uniform_partition(a, b, n), a, b, n.suc, i)
                        interval_contains(a, b, uniform_partition(a, b, n, i))
                        partition_point_in_interval(uniform_partition(a, b, n), a, b, n.suc, i + 1)
                        interval_contains(a, b, uniform_partition(a, b, n, i + 1))
                        interval_contains_mono(a, b, uniform_partition(a, b, n, i),
                            uniform_partition(a, b, n, i + 1), t)
                        interval_contains(a, b, t)
                        forall(t0: Real) {
                            interval_contains(a, b, t0) implies lb <= f(t0)
                        }
                        interval_contains(a, b, t) implies lb <= f(t)
                        lb <= f(t)
                    }
                }
                image_lower_bound(f, uniform_partition(a, b, n, i), uniform_partition(a, b, n, i + 1), lb)
                is_set_lower_bound(interval_image(f, uniform_partition(a, b, n, i),
                    uniform_partition(a, b, n, i + 1)), lb)
                exists(b0: Real) {
                    is_set_lower_bound(interval_image(f, uniform_partition(a, b, n, i),
                        uniform_partition(a, b, n, i + 1)), b0)
                }
                has_lower_bound(interval_image(f, uniform_partition(a, b, n, i),
                    uniform_partition(a, b, n, i + 1)))
                forall(u: Real, v: Real) {
                    if interval_contains(uniform_partition(a, b, n, i), uniform_partition(a, b, n, i + 1), u) and
                       interval_contains(uniform_partition(a, b, n, i), uniform_partition(a, b, n, i + 1), v) {
                        interval_contains_left(uniform_partition(a, b, n, i), uniform_partition(a, b, n, i + 1), u)
                        uniform_partition(a, b, n, i) <= u
                        interval_contains_right(uniform_partition(a, b, n, i), uniform_partition(a, b, n, i + 1), u)
                        u <= uniform_partition(a, b, n, i + 1)
                        interval_contains_left(uniform_partition(a, b, n, i), uniform_partition(a, b, n, i + 1), v)
                        uniform_partition(a, b, n, i) <= v
                        interval_contains_right(uniform_partition(a, b, n, i), uniform_partition(a, b, n, i + 1), v)
                        v <= uniform_partition(a, b, n, i + 1)
                        interval_abs_diff_le_width(uniform_partition(a, b, n, i),
                            uniform_partition(a, b, n, i + 1), u, v)
                        (u - v).abs <= uniform_partition(a, b, n, i + 1) - uniform_partition(a, b, n, i)
                        (u - v).abs <= uniform_mesh(a, b, n)
                        forall(u0: Real, v0: Real) {
                            interval_contains(a, b, u0) and interval_contains(a, b, v0) implies (f(u0) - f(v0)).abs <= m * (u0 - v0).abs
                        }
                        interval_contains(a, b, u) and interval_contains(a, b, v) implies (f(u) - f(v)).abs <= m * (u - v).abs
                        partition_point_in_interval(uniform_partition(a, b, n), a, b, n.suc, i)
                        interval_contains(a, b, uniform_partition(a, b, n, i))
                        partition_point_in_interval(uniform_partition(a, b, n), a, b, n.suc, i + 1)
                        interval_contains(a, b, uniform_partition(a, b, n, i + 1))
                        interval_contains_mono(a, b, uniform_partition(a, b, n, i),
                            uniform_partition(a, b, n, i + 1), u)
                        interval_contains(a, b, u)
                        interval_contains_mono(a, b, uniform_partition(a, b, n, i),
                            uniform_partition(a, b, n, i + 1), v)
                        interval_contains(a, b, v)
                        interval_contains(a, b, u) and interval_contains(a, b, v)
                        (f(u) - f(v)).abs <= m * (u - v).abs
                        Real.0 <= m
                        mul_le_mul_of_nonneg_left((u - v).abs, uniform_mesh(a, b, n), m)
                        m * (u - v).abs <= m * uniform_mesh(a, b, n)
                        lte_trans[Real]((f(u) - f(v)).abs, m * (u - v).abs, m * uniform_mesh(a, b, n))
                        (f(u) - f(v)).abs <= m * uniform_mesh(a, b, n)
                        (f(u) - f(v)).abs <= m * (uniform_partition(a, b, n, i + 1) - uniform_partition(a, b, n, i))
                    }
                }
                interval_sup_sub_inf_le_lipschitz_m(f, m,
                    uniform_partition(a, b, n, i), uniform_partition(a, b, n, i + 1))
                interval_sup(f, uniform_partition(a, b, n, i), uniform_partition(a, b, n, i + 1)) -
                    interval_inf(f, uniform_partition(a, b, n, i), uniform_partition(a, b, n, i + 1)) <= m * (uniform_partition(a, b, n, i + 1) - uniform_partition(a, b, n, i))
                interval_sup(f, uniform_partition(a, b, n, i), uniform_partition(a, b, n, i + 1)) -
                    interval_inf(f, uniform_partition(a, b, n, i), uniform_partition(a, b, n, i + 1)) <= m * uniform_mesh(a, b, n)
                partition_step_upper(f, uniform_partition(a, b, n), i) =
                    interval_sup(f, uniform_partition(a, b, n, i), uniform_partition(a, b, n, i + 1)) *
                        diff_step(uniform_partition(a, b, n), i)
                diff_step(uniform_partition(a, b, n), i) =
                    uniform_partition(a, b, n, i + 1) - uniform_partition(a, b, n, i)
                diff_step(uniform_partition(a, b, n), i) = uniform_mesh(a, b, n)
                partition_step_upper(f, uniform_partition(a, b, n), i) =
                    interval_sup(f, uniform_partition(a, b, n, i), uniform_partition(a, b, n, i + 1)) *
                        uniform_mesh(a, b, n)
                partition_step_lower(f, uniform_partition(a, b, n), i) =
                    interval_inf(f, uniform_partition(a, b, n, i), uniform_partition(a, b, n, i + 1)) *
                        diff_step(uniform_partition(a, b, n), i)
                partition_step_lower(f, uniform_partition(a, b, n), i) =
                    interval_inf(f, uniform_partition(a, b, n, i), uniform_partition(a, b, n, i + 1)) *
                        uniform_mesh(a, b, n)
                partition_step_upper(f, uniform_partition(a, b, n), i) -
                    partition_step_lower(f, uniform_partition(a, b, n), i) =
                    (interval_sup(f, uniform_partition(a, b, n, i), uniform_partition(a, b, n, i + 1)) -
                        interval_inf(f, uniform_partition(a, b, n, i), uniform_partition(a, b, n, i + 1))) *
                        uniform_mesh(a, b, n)
                from_nat_suc_pos_real(n)
                from_nat[Real](n.suc) > Real.0
                from_nat[Real](n.suc) != Real.0
                sub_nonneg(a, b)
                Real.0 <= b - a
                div_nonneg_pos_denom(b - a, from_nat[Real](n.suc))
                Real.0 <= uniform_mesh(a, b, n)
                mul_le_mul_of_nonneg_right(
                    interval_sup(f, uniform_partition(a, b, n, i), uniform_partition(a, b, n, i + 1)) -
                        interval_inf(f, uniform_partition(a, b, n, i), uniform_partition(a, b, n, i + 1)),
                    m * uniform_mesh(a, b, n), uniform_mesh(a, b, n))
                (interval_sup(f, uniform_partition(a, b, n, i), uniform_partition(a, b, n, i + 1)) -
                    interval_inf(f, uniform_partition(a, b, n, i), uniform_partition(a, b, n, i + 1))) *
                    uniform_mesh(a, b, n) <= (m * uniform_mesh(a, b, n)) * uniform_mesh(a, b, n)
                (m * uniform_mesh(a, b, n)) * uniform_mesh(a, b, n) =
                    m * (uniform_mesh(a, b, n) * uniform_mesh(a, b, n))
                (interval_sup(f, uniform_partition(a, b, n, i), uniform_partition(a, b, n, i + 1)) -
                    interval_inf(f, uniform_partition(a, b, n, i), uniform_partition(a, b, n, i + 1))) *
                    uniform_mesh(a, b, n) <= m * (uniform_mesh(a, b, n) * uniform_mesh(a, b, n))
                partition_step_upper(f, uniform_partition(a, b, n), i) -
                    partition_step_lower(f, uniform_partition(a, b, n), i) <= m * (uniform_mesh(a, b, n) * uniform_mesh(a, b, n))
                sub_seq(partition_step_upper(f, uniform_partition(a, b, n)),
                    partition_step_lower(f, uniform_partition(a, b, n)), i) =
                    partition_step_upper(f, uniform_partition(a, b, n), i) -
                    partition_step_lower(f, uniform_partition(a, b, n), i)
                sub_seq(partition_step_upper(f, uniform_partition(a, b, n)),
                    partition_step_lower(f, uniform_partition(a, b, n)), i) <= m * (uniform_mesh(a, b, n) * uniform_mesh(a, b, n))
            }
        }
        partial_lte(sub_seq(partition_step_upper(f, uniform_partition(a, b, n)),
                partition_step_lower(f, uniform_partition(a, b, n))),
            const_seq(m * (uniform_mesh(a, b, n) * uniform_mesh(a, b, n))), n.suc)
        partial(sub_seq(partition_step_upper(f, uniform_partition(a, b, n)),
                partition_step_lower(f, uniform_partition(a, b, n))), n.suc) <= partial(const_seq(m * (uniform_mesh(a, b, n) * uniform_mesh(a, b, n))), n.suc)
        forall(j: Nat) {
            if j < n.suc {
                const_seq(m * (uniform_mesh(a, b, n) * uniform_mesh(a, b, n)), j) =
                    m * (uniform_mesh(a, b, n) * uniform_mesh(a, b, n))
            }
        }
        partial_const(const_seq(m * (uniform_mesh(a, b, n) * uniform_mesh(a, b, n))), n.suc,
            m * (uniform_mesh(a, b, n) * uniform_mesh(a, b, n)))
        partial(const_seq(m * (uniform_mesh(a, b, n) * uniform_mesh(a, b, n))), n.suc) =
            from_nat[Real](n.suc) * (m * (uniform_mesh(a, b, n) * uniform_mesh(a, b, n)))
        partial(sub_seq(partition_step_upper(f, uniform_partition(a, b, n)),
                partition_step_lower(f, uniform_partition(a, b, n))), n.suc) <= from_nat[Real](n.suc) * (m * (uniform_mesh(a, b, n) * uniform_mesh(a, b, n)))
        partial_sub_seq(partition_step_upper(f, uniform_partition(a, b, n)),
            partition_step_lower(f, uniform_partition(a, b, n)), n.suc)
        partial(sub_seq(partition_step_upper(f, uniform_partition(a, b, n)),
                partition_step_lower(f, uniform_partition(a, b, n))), n.suc) =
            partial(partition_step_upper(f, uniform_partition(a, b, n)), n.suc) -
            partial(partition_step_lower(f, uniform_partition(a, b, n)), n.suc)
        partial(partition_step_upper(f, uniform_partition(a, b, n)), n.suc) -
            partial(partition_step_lower(f, uniform_partition(a, b, n)), n.suc) <= from_nat[Real](n.suc) * (m * (uniform_mesh(a, b, n) * uniform_mesh(a, b, n)))
        from_nat_suc_pos_real(n)
        from_nat[Real](n.suc) > Real.0
        from_nat[Real](n.suc) != Real.0
        div_mul_cancel_denominator(b - a, from_nat[Real](n.suc))
        ((b - a) / from_nat[Real](n.suc)) * from_nat[Real](n.suc) = b - a
        uniform_mesh(a, b, n) * from_nat[Real](n.suc) = b - a
        from_nat[Real](n.suc) * (uniform_mesh(a, b, n) * uniform_mesh(a, b, n)) =
            (from_nat[Real](n.suc) * uniform_mesh(a, b, n)) * uniform_mesh(a, b, n)
        from_nat[Real](n.suc) * uniform_mesh(a, b, n) = b - a
        from_nat[Real](n.suc) * (uniform_mesh(a, b, n) * uniform_mesh(a, b, n)) =
            (b - a) * uniform_mesh(a, b, n)
        mul_frac_right(b - a, from_nat[Real](n.suc), b - a)
        ((b - a) / from_nat[Real](n.suc)) * (b - a) = ((b - a) * (b - a)) / from_nat[Real](n.suc)
        (b - a) * uniform_mesh(a, b, n) = ((b - a) * (b - a)) / from_nat[Real](n.suc)
        from_nat[Real](n.suc) * (uniform_mesh(a, b, n) * uniform_mesh(a, b, n)) =
            ((b - a) * (b - a)) / from_nat[Real](n.suc)
        from_nat[Real](n.suc) * (m * (uniform_mesh(a, b, n) * uniform_mesh(a, b, n))) =
            m * (from_nat[Real](n.suc) * (uniform_mesh(a, b, n) * uniform_mesh(a, b, n)))
        from_nat[Real](n.suc) * (m * (uniform_mesh(a, b, n) * uniform_mesh(a, b, n))) =
            m * (((b - a) * (b - a)) / from_nat[Real](n.suc))
        partial(partition_step_upper(f, uniform_partition(a, b, n)), n.suc) -
            partial(partition_step_lower(f, uniform_partition(a, b, n)), n.suc) <= m * (((b - a) * (b - a)) / from_nat[Real](n.suc))
        upper_sum(f, uniform_partition(a, b, n), n.suc) =
            partial(partition_step_upper(f, uniform_partition(a, b, n)), n.suc)
        lower_sum(f, uniform_partition(a, b, n), n.suc) =
            partial(partition_step_lower(f, uniform_partition(a, b, n)), n.suc)
        upper_sum(f, uniform_partition(a, b, n), n.suc) -
            lower_sum(f, uniform_partition(a, b, n), n.suc) <= m * (((b - a) * (b - a)) / from_nat[Real](n.suc))
    }
}

/// An m-Lipschitz function on [a, b] bounded there, with known continuous
/// antiderivative, is integrable on [a, b].
theorem fn_integrable_gen(f: Real -> Real, g: Real -> Real, a: Real, b: Real, m: Real, lb: Real, ub: Real) {
    a <= b and Real.0 <= m and continuous(g) and is_derivative_fn(g, f) and
    (forall(u: Real, v: Real) {
        interval_contains(a, b, u) and interval_contains(a, b, v) implies (f(u) - f(v)).abs <= m * (u - v).abs
    }) and
    (forall(t: Real) { interval_contains(a, b, t) implies lb <= f(t) }) and
    (forall(t: Real) { interval_contains(a, b, t) implies f(t) <= ub }) implies
    is_integrable(f, a, b)
} by {
    if a <= b and Real.0 <= m and continuous(g) and is_derivative_fn(g, f) and
       (forall(u: Real, v: Real) {
           interval_contains(a, b, u) and interval_contains(a, b, v) implies (f(u) - f(v)).abs <= m * (u - v).abs
       }) and
       (forall(t: Real) { interval_contains(a, b, t) implies lb <= f(t) }) and
       (forall(t: Real) { interval_contains(a, b, t) implies f(t) <= ub }) {
        fn_lower_sum_set_sup_exists(f, g, a, b, lb)
        let l: Real satisfy {
            is_set_supremum(lower_sum_set(f, a, b), l)
        }
        fn_upper_sum_set_inf_exists(f, g, a, b, ub)
        let u: Real satisfy {
            is_set_infimum(upper_sum_set(f, a, b), u)
        }
        fn_lower_sum_set_bounded_above(f, g, a, b, lb)
        is_set_upper_bound(lower_sum_set(f, a, b), g(b) - g(a))
        set_supremum_le_upper_bound(lower_sum_set(f, a, b), l, g(b) - g(a))
        l <= g(b) - g(a)
        fn_upper_sum_set_bounded_below(f, g, a, b, ub)
        is_set_lower_bound(upper_sum_set(f, a, b), g(b) - g(a))
        set_lower_bound_le_infimum(upper_sum_set(f, a, b), u, g(b) - g(a))
        g(b) - g(a) <= u
        lte_trans[Real](l, g(b) - g(a), u)
        l <= u
        sub_nonneg(l, u)
        Real.0 <= u - l
        forall(n: Nat) {
            lipschitz_uniform_upper_minus_lower_m(f, m, a, b, n, lb, ub)
            upper_sum(f, uniform_partition(a, b, n), n.suc) -
                lower_sum(f, uniform_partition(a, b, n), n.suc) <= m * (((b - a) * (b - a)) / from_nat[Real](n.suc))
            uniform_partition_is_partition(a, b, n)
            is_partition(uniform_partition(a, b, n), a, b, n.suc)
            is_partition(uniform_partition(a, b, n), a, b, n.suc) and
                lower_sum(f, uniform_partition(a, b, n), n.suc) =
                    lower_sum(f, uniform_partition(a, b, n), n.suc)
            exists(p0: Nat -> Real, n0: Nat) {
                is_partition(p0, a, b, n0) and
                    lower_sum(f, uniform_partition(a, b, n), n.suc) = lower_sum(f, p0, n0)
            }
            lower_sum_contains(f, a, b, lower_sum(f, uniform_partition(a, b, n), n.suc))
            lower_sum_set(f, a, b).contains(lower_sum(f, uniform_partition(a, b, n), n.suc)) =
                lower_sum_contains(f, a, b, lower_sum(f, uniform_partition(a, b, n), n.suc))
            lower_sum_set(f, a, b).contains(lower_sum(f, uniform_partition(a, b, n), n.suc))
            set_member_le_supremum(lower_sum_set(f, a, b), l,
                lower_sum(f, uniform_partition(a, b, n), n.suc))
            lower_sum(f, uniform_partition(a, b, n), n.suc) <= l
            is_partition(uniform_partition(a, b, n), a, b, n.suc) and
                upper_sum(f, uniform_partition(a, b, n), n.suc) =
                    upper_sum(f, uniform_partition(a, b, n), n.suc)
            exists(p1: Nat -> Real, n1: Nat) {
                is_partition(p1, a, b, n1) and
                    upper_sum(f, uniform_partition(a, b, n), n.suc) = upper_sum(f, p1, n1)
            }
            upper_sum_contains(f, a, b, upper_sum(f, uniform_partition(a, b, n), n.suc))
            upper_sum_set(f, a, b).contains(upper_sum(f, uniform_partition(a, b, n), n.suc)) =
                upper_sum_contains(f, a, b, upper_sum(f, uniform_partition(a, b, n), n.suc))
            upper_sum_set(f, a, b).contains(upper_sum(f, uniform_partition(a, b, n), n.suc))
            set_infimum_is_lower_bound(upper_sum_set(f, a, b), u)
            is_set_lower_bound(upper_sum_set(f, a, b), u)
            set_lower_bound_contains_le(upper_sum_set(f, a, b), u,
                upper_sum(f, uniform_partition(a, b, n), n.suc))
            u <= upper_sum(f, uniform_partition(a, b, n), n.suc)
            neg_lte_flip(lower_sum(f, uniform_partition(a, b, n), n.suc), l)
            -l <= -lower_sum(f, uniform_partition(a, b, n), n.suc)
            add_le_add(u, upper_sum(f, uniform_partition(a, b, n), n.suc),
                -l, -lower_sum(f, uniform_partition(a, b, n), n.suc))
            u + -l <= upper_sum(f, uniform_partition(a, b, n), n.suc) +
                -lower_sum(f, uniform_partition(a, b, n), n.suc)
            u - l = u + -l
            upper_sum(f, uniform_partition(a, b, n), n.suc) -
                lower_sum(f, uniform_partition(a, b, n), n.suc) =
                upper_sum(f, uniform_partition(a, b, n), n.suc) +
                -lower_sum(f, uniform_partition(a, b, n), n.suc)
            u - l <= upper_sum(f, uniform_partition(a, b, n), n.suc) -
                lower_sum(f, uniform_partition(a, b, n), n.suc)
            lte_trans[Real](u - l,
                upper_sum(f, uniform_partition(a, b, n), n.suc) -
                    lower_sum(f, uniform_partition(a, b, n), n.suc),
                m * (((b - a) * (b - a)) / from_nat[Real](n.suc)))
            u - l <= m * (((b - a) * (b - a)) / from_nat[Real](n.suc))
            m * (((b - a) * (b - a)) / from_nat[Real](n.suc)) =
                (m * ((b - a) * (b - a))) / from_nat[Real](n.suc)
            u - l <= (m * ((b - a) * (b - a))) / from_nat[Real](n.suc)
        }
        nonneg_frac_all_n_imp_zero(m * ((b - a) * (b - a)), u - l)
        u - l = Real.0
        sub_zero_imp_eq(u, l)
        u = l
        is_set_infimum(upper_sum_set(f, a, b), l)
        is_set_supremum(lower_sum_set(f, a, b), l) and is_set_infimum(upper_sum_set(f, a, b), l)
        exists(m1: Real) {
            is_set_supremum(lower_sum_set(f, a, b), m1) and is_set_infimum(upper_sum_set(f, a, b), m1)
        }
        is_integrable(f, a, b)
    }
}
