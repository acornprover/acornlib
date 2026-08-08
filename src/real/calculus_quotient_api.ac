/// Quotient and reciprocal extensions for the real calculus API.

from data.basic.function_algebra import pointwise_add, pointwise_neg, pointwise_mul
from real.real_base import Real
from real.derivative_basic import has_derivative_at, differentiable_at
from real.calculus_api import is_derivative_fn, differentiable_everywhere,
    is_derivative_fn_at, differentiable_everywhere_at
from real.derivative_quotient import reciprocal_real, pointwise_div_real,
    pointwise_reciprocal_real, reciprocal_real_apply, pointwise_div_real_apply,
    pointwise_reciprocal_real_apply, pointwise_reciprocal_real_eq_compose,
    pointwise_div_real_eq_mul_reciprocal, pointwise_div_const_eq_mul_inverse,
    derivative_reciprocal_real, derivative_pointwise_reciprocal_real,
    differentiable_pointwise_reciprocal_real, derivative_pointwise_div,
    differentiable_pointwise_div, derivative_pointwise_div_from_reciprocal,
    differentiable_pointwise_div_from_reciprocal, derivative_pointwise_div_const,
    differentiable_pointwise_div_const
from algebra.field.field import inverse_not_zero, mul_not_zero

/// A real-valued function is nowhere zero.
define nonvanishing_everywhere(f: Real -> Real) -> Bool {
    forall(x: Real) {
        f(x) != Real.0
    }
}

/// Instantiating a nonvanishing-everywhere hypothesis at a single point.
theorem nonvanishing_everywhere_at(f: Real -> Real, x: Real) {
    nonvanishing_everywhere(f) implies f(x) != Real.0
} by {
    if nonvanishing_everywhere(f) {
        nonvanishing_everywhere(f) = forall(y: Real) {
            f(y) != Real.0
        }
        f(x) != Real.0
    }
}

/// A nonzero constant function is nonvanishing everywhere.
theorem nonvanishing_everywhere_constant(c: Real) {
    c != Real.0 implies nonvanishing_everywhere(constant[Real, Real](c))
} by {
    if c != Real.0 {
        forall(x: Real) {
            constant[Real, Real](c, x) = c
            constant[Real, Real](c, x) != Real.0
        }
    }
}

/// The reciprocal of a nonzero real is nonzero.
theorem reciprocal_real_nonzero(x: Real) {
    x != Real.0 implies reciprocal_real(x) != Real.0
} by {
    if x != Real.0 {
        reciprocal_real_apply(x)
        reciprocal_real(x) = x.inverse
        inverse_not_zero[Real](x)
        x.inverse != Real.0
        reciprocal_real(x) != Real.0
    }
}

/// The pointwise reciprocal of a nonvanishing function is nonvanishing.
theorem nonvanishing_everywhere_reciprocal(f: Real -> Real) {
    nonvanishing_everywhere(f) implies nonvanishing_everywhere(pointwise_reciprocal_real(f))
} by {
    if nonvanishing_everywhere(f) {
        forall(x: Real) {
            nonvanishing_everywhere_at(f, x)
            f(x) != Real.0
            pointwise_reciprocal_real_apply(f, x)
            pointwise_reciprocal_real(f, x) = f(x).inverse
            inverse_not_zero[Real](f(x))
            f(x).inverse != Real.0
            pointwise_reciprocal_real(f, x) != Real.0
        }
    }
}

/// A pointwise product of nonvanishing functions is nonvanishing.
theorem nonvanishing_everywhere_mul(f: Real -> Real, g: Real -> Real) {
    nonvanishing_everywhere(f) and nonvanishing_everywhere(g)
        implies nonvanishing_everywhere(pointwise_mul(f, g))
} by {
    if nonvanishing_everywhere(f) and nonvanishing_everywhere(g) {
        forall(x: Real) {
            nonvanishing_everywhere_at(f, x)
            nonvanishing_everywhere_at(g, x)
            f(x) != Real.0
            g(x) != Real.0
            pointwise_mul(f, g, x) = f(x) * g(x)
            mul_not_zero[Real](f(x), g(x))
            f(x) * g(x) != Real.0
            pointwise_mul(f, g, x) != Real.0
        }
    }
}

/// A pointwise quotient of nonvanishing functions is nonvanishing.
theorem nonvanishing_everywhere_div(f: Real -> Real, g: Real -> Real) {
    nonvanishing_everywhere(f) and nonvanishing_everywhere(g)
        implies nonvanishing_everywhere(pointwise_div_real(f, g))
} by {
    if nonvanishing_everywhere(f) and nonvanishing_everywhere(g) {
        forall(x: Real) {
            nonvanishing_everywhere_at(f, x)
            nonvanishing_everywhere_at(g, x)
            f(x) != Real.0
            g(x) != Real.0
            inverse_not_zero[Real](g(x))
            g(x).inverse != Real.0
            pointwise_div_real(f, g, x) = f(x) / g(x)
            f(x) / g(x) = f(x) * g(x).inverse
            mul_not_zero[Real](f(x), g(x).inverse)
            f(x) * g(x).inverse != Real.0
            pointwise_div_real(f, g, x) != Real.0
        }
    }
}

/// The ordinary reciprocal function has its standard derivative at any nonzero point.
theorem reciprocal_real_has_derivative_at(x: Real) {
    x != Real.0 implies has_derivative_at(reciprocal_real, x, -Real.1 / (x * x))
} by {
    if x != Real.0 {
        derivative_reciprocal_real(x)
        has_derivative_at(reciprocal_real, x, -Real.1 / (x * x))
    }
}

/// The pointwise reciprocal rule packaged for API users.
theorem reciprocal_has_derivative_at(
    g: Real -> Real, x: Real, dg: Real
) {
    has_derivative_at(g, x, dg) and g(x) != Real.0 implies
        has_derivative_at(pointwise_reciprocal_real(g), x, (-Real.1 / (g(x) * g(x))) * dg)
} by {
    if has_derivative_at(g, x, dg) and g(x) != Real.0 {
        derivative_pointwise_reciprocal_real(g, x, dg)
        has_derivative_at(pointwise_reciprocal_real(g), x, (-Real.1 / (g(x) * g(x))) * dg)
    }
}

/// Differentiability of pointwise reciprocals packaged for API users.
theorem reciprocal_differentiable_at(g: Real -> Real, x: Real) {
    differentiable_at(g, x) and g(x) != Real.0
        implies differentiable_at(pointwise_reciprocal_real(g), x)
} by {
    if differentiable_at(g, x) and g(x) != Real.0 {
        differentiable_pointwise_reciprocal_real(g, x)
        differentiable_at(pointwise_reciprocal_real(g), x)
    }
}

/// The quotient rule packaged for API users.
theorem quotient_has_derivative_at(
    f: Real -> Real, g: Real -> Real, x: Real, df: Real, dg: Real
) {
    has_derivative_at(f, x, df) and has_derivative_at(g, x, dg) and g(x) != Real.0 implies
        has_derivative_at(pointwise_div_real(f, g), x, (df * g(x) - f(x) * dg) / (g(x) * g(x)))
} by {
    if has_derivative_at(f, x, df) and has_derivative_at(g, x, dg) and g(x) != Real.0 {
        derivative_pointwise_div(f, g, x, df, dg)
        has_derivative_at(pointwise_div_real(f, g), x, (df * g(x) - f(x) * dg) / (g(x) * g(x)))
    }
}

/// Differentiability of pointwise quotients packaged for API users.
theorem quotient_differentiable_at(f: Real -> Real, g: Real -> Real, x: Real) {
    differentiable_at(f, x) and differentiable_at(g, x) and g(x) != Real.0
        implies differentiable_at(pointwise_div_real(f, g), x)
} by {
    if differentiable_at(f, x) and differentiable_at(g, x) and g(x) != Real.0 {
        differentiable_pointwise_div(f, g, x)
        differentiable_at(pointwise_div_real(f, g), x)
    }
}

/// Division by a constant preserves derivatives.
theorem div_const_has_derivative_at(f: Real -> Real, c: Real, x: Real, d: Real) {
    has_derivative_at(f, x, d) implies has_derivative_at(pointwise_div_real(f, constant[Real, Real](c)), x, d / c)
} by {
    if has_derivative_at(f, x, d) {
        derivative_pointwise_div_const(f, c, x, d)
        has_derivative_at(pointwise_div_real(f, constant[Real, Real](c)), x, d / c)
    }
}

/// Division by a constant preserves differentiability.
theorem div_const_differentiable_at(f: Real -> Real, c: Real, x: Real) {
    differentiable_at(f, x) implies differentiable_at(pointwise_div_real(f, constant[Real, Real](c)), x)
} by {
    if differentiable_at(f, x) {
        differentiable_pointwise_div_const(f, c, x)
        differentiable_at(pointwise_div_real(f, constant[Real, Real](c)), x)
    }
}

/// A global derivative for a nonvanishing function transports through reciprocal.
theorem derivative_fn_reciprocal(g: Real -> Real, dg: Real -> Real) {
    is_derivative_fn(g, dg) and nonvanishing_everywhere(g) implies
        is_derivative_fn(
            pointwise_reciprocal_real(g),
            pointwise_mul(pointwise_div_real(constant[Real, Real](-Real.1), pointwise_mul(g, g)), dg)
        )
} by {
    if is_derivative_fn(g, dg) and nonvanishing_everywhere(g) {
        forall(x: Real) {
            is_derivative_fn_at(g, dg, x)
            nonvanishing_everywhere_at(g, x)
            g(x) != Real.0
            derivative_pointwise_reciprocal_real(g, x, dg(x))
            has_derivative_at(pointwise_reciprocal_real(g), x, (-Real.1 / (g(x) * g(x))) * dg(x))
            constant[Real, Real](-Real.1, x) = -Real.1
            pointwise_mul(g, g, x) = g(x) * g(x)
            pointwise_div_real(constant[Real, Real](-Real.1), pointwise_mul(g, g), x) =
                constant[Real, Real](-Real.1, x) / pointwise_mul(g, g, x)
            pointwise_div_real(constant[Real, Real](-Real.1), pointwise_mul(g, g), x) =
                -Real.1 / (g(x) * g(x))
            pointwise_mul(pointwise_div_real(constant[Real, Real](-Real.1), pointwise_mul(g, g)), dg, x) =
                pointwise_div_real(constant[Real, Real](-Real.1), pointwise_mul(g, g), x) * dg(x)
            pointwise_mul(pointwise_div_real(constant[Real, Real](-Real.1), pointwise_mul(g, g)), dg, x) =
                (-Real.1 / (g(x) * g(x))) * dg(x)
            has_derivative_at(
                pointwise_reciprocal_real(g),
                x,
                pointwise_mul(pointwise_div_real(constant[Real, Real](-Real.1), pointwise_mul(g, g)), dg, x)
            )
        }
    }
}

/// A global quotient rule for nonvanishing denominators.
theorem derivative_fn_quotient(
    f: Real -> Real, g: Real -> Real, df: Real -> Real, dg: Real -> Real
) {
    is_derivative_fn(f, df) and is_derivative_fn(g, dg) and nonvanishing_everywhere(g) implies
        is_derivative_fn(
            pointwise_div_real(f, g),
            pointwise_div_real(
                pointwise_add(pointwise_mul(df, g), pointwise_neg(pointwise_mul(f, dg))),
                pointwise_mul(g, g)
            )
        )
} by {
    if is_derivative_fn(f, df) and is_derivative_fn(g, dg) and nonvanishing_everywhere(g) {
        forall(x: Real) {
            is_derivative_fn_at(f, df, x)
            is_derivative_fn_at(g, dg, x)
            nonvanishing_everywhere_at(g, x)
            g(x) != Real.0
            derivative_pointwise_div(f, g, x, df(x), dg(x))
            has_derivative_at(pointwise_div_real(f, g), x, (df(x) * g(x) - f(x) * dg(x)) / (g(x) * g(x)))
            pointwise_mul(df, g, x) = df(x) * g(x)
            pointwise_mul(f, dg, x) = f(x) * dg(x)
            pointwise_neg(pointwise_mul(f, dg), x) = -pointwise_mul(f, dg, x)
            pointwise_neg(pointwise_mul(f, dg), x) = -(f(x) * dg(x))
            pointwise_add(pointwise_mul(df, g), pointwise_neg(pointwise_mul(f, dg)), x) =
                pointwise_mul(df, g, x) + pointwise_neg(pointwise_mul(f, dg), x)
            pointwise_add(pointwise_mul(df, g), pointwise_neg(pointwise_mul(f, dg)), x) =
                df(x) * g(x) + -(f(x) * dg(x))
            df(x) * g(x) + -(f(x) * dg(x)) = df(x) * g(x) - f(x) * dg(x)
            pointwise_add(pointwise_mul(df, g), pointwise_neg(pointwise_mul(f, dg)), x) =
                df(x) * g(x) - f(x) * dg(x)
            pointwise_mul(g, g, x) = g(x) * g(x)
            pointwise_div_real(
                pointwise_add(pointwise_mul(df, g), pointwise_neg(pointwise_mul(f, dg))),
                pointwise_mul(g, g),
                x
            ) = pointwise_add(pointwise_mul(df, g), pointwise_neg(pointwise_mul(f, dg)), x) /
                pointwise_mul(g, g, x)
            pointwise_div_real(
                pointwise_add(pointwise_mul(df, g), pointwise_neg(pointwise_mul(f, dg))),
                pointwise_mul(g, g),
                x
            ) = (df(x) * g(x) - f(x) * dg(x)) / (g(x) * g(x))
            has_derivative_at(
                pointwise_div_real(f, g),
                x,
                pointwise_div_real(
                    pointwise_add(pointwise_mul(df, g), pointwise_neg(pointwise_mul(f, dg))),
                    pointwise_mul(g, g),
                    x
                )
            )
        }
    }
}

/// A global derivative transports through division by a constant.
theorem derivative_fn_div_const(f: Real -> Real, df: Real -> Real, c: Real) {
    is_derivative_fn(f, df) implies
        is_derivative_fn(pointwise_div_real(f, constant[Real, Real](c)), pointwise_div_real(df, constant[Real, Real](c)))
} by {
    if is_derivative_fn(f, df) {
        forall(x: Real) {
            is_derivative_fn_at(f, df, x)
            derivative_pointwise_div_const(f, c, x, df(x))
            has_derivative_at(pointwise_div_real(f, constant[Real, Real](c)), x, df(x) / c)
            constant[Real, Real](c, x) = c
            pointwise_div_real(df, constant[Real, Real](c), x) = df(x) / constant[Real, Real](c, x)
            pointwise_div_real(df, constant[Real, Real](c), x) = df(x) / c
            has_derivative_at(
                pointwise_div_real(f, constant[Real, Real](c)),
                x,
                pointwise_div_real(df, constant[Real, Real](c), x)
            )
        }
    }
}

/// Global differentiability is preserved by reciprocal on nonvanishing functions.
theorem differentiable_everywhere_reciprocal(g: Real -> Real) {
    differentiable_everywhere(g) and nonvanishing_everywhere(g)
        implies differentiable_everywhere(pointwise_reciprocal_real(g))
} by {
    if differentiable_everywhere(g) and nonvanishing_everywhere(g) {
        forall(x: Real) {
            differentiable_everywhere_at(g, x)
            nonvanishing_everywhere_at(g, x)
            differentiable_at(g, x)
            g(x) != Real.0
            differentiable_pointwise_reciprocal_real(g, x)
            differentiable_at(pointwise_reciprocal_real(g), x)
        }
    }
}

/// Global differentiability is preserved by quotients with nonvanishing denominators.
theorem differentiable_everywhere_quotient(f: Real -> Real, g: Real -> Real) {
    differentiable_everywhere(f) and differentiable_everywhere(g) and nonvanishing_everywhere(g)
        implies differentiable_everywhere(pointwise_div_real(f, g))
} by {
    if differentiable_everywhere(f) and differentiable_everywhere(g) and nonvanishing_everywhere(g) {
        forall(x: Real) {
            differentiable_everywhere_at(f, x)
            differentiable_everywhere_at(g, x)
            nonvanishing_everywhere_at(g, x)
            differentiable_at(f, x)
            differentiable_at(g, x)
            g(x) != Real.0
            differentiable_pointwise_div(f, g, x)
            differentiable_at(pointwise_div_real(f, g), x)
        }
    }
}

/// Global differentiability is preserved by division by a constant.
theorem differentiable_everywhere_div_const(f: Real -> Real, c: Real) {
    differentiable_everywhere(f) implies differentiable_everywhere(pointwise_div_real(f, constant[Real, Real](c)))
} by {
    if differentiable_everywhere(f) {
        forall(x: Real) {
            differentiable_everywhere_at(f, x)
            differentiable_at(f, x)
            differentiable_pointwise_div_const(f, c, x)
            differentiable_at(pointwise_div_real(f, constant[Real, Real](c)), x)
        }
    }
}
