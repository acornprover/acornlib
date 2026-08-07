from data.basic.function_algebra import pointwise_add, pointwise_mul
from data.basic.functions import compose, identity_fn
from real.continuity_cube import cube_real
from real.continuity_square import square_real
from real.derivative_basic import has_derivative_at, differentiable_at
from real.derivative_chain_corollaries import derivative_affine_after,
    derivative_after_affine, differentiable_affine_after,
    differentiable_after_affine
from real.derivative_polynomial_chain import cube_real_differentiable_at,
    cube_real_has_derivative_at, square_real_differentiable_at,
    square_real_has_derivative_at
from real.real_base import Real

/// Composing the square function after an affine function follows the chain rule.
theorem derivative_square_real_after_affine(c: Real, b: Real, x0: Real) {
    has_derivative_at(
        compose(square_real, pointwise_add(pointwise_mul(constant[Real, Real](c), identity_fn[Real]), constant[Real, Real](b))),
        x0,
        (
            pointwise_add(pointwise_mul(constant[Real, Real](c), identity_fn[Real]), constant[Real, Real](b), x0) * Real.1 +
            pointwise_add(pointwise_mul(constant[Real, Real](c), identity_fn[Real]), constant[Real, Real](b), x0) * Real.1
        ) * c
    )
} by {
    let affine = pointwise_add(pointwise_mul(constant[Real, Real](c), identity_fn[Real]), constant[Real, Real](b))
    square_real_has_derivative_at(affine(x0))
    derivative_after_affine(c, b, square_real, x0, affine(x0) * Real.1 + affine(x0) * Real.1)
    has_derivative_at(
        compose(square_real, pointwise_add(pointwise_mul(constant[Real, Real](c), identity_fn[Real]), constant[Real, Real](b))),
        x0,
        (affine(x0) * Real.1 + affine(x0) * Real.1) * c
    )
}

/// Composing the cube function after an affine function follows the chain rule.
theorem derivative_cube_real_after_affine(c: Real, b: Real, x0: Real) {
    has_derivative_at(
        compose(cube_real, pointwise_add(pointwise_mul(constant[Real, Real](c), identity_fn[Real]), constant[Real, Real](b))),
        x0,
        (
            square_real(pointwise_add(pointwise_mul(constant[Real, Real](c), identity_fn[Real]), constant[Real, Real](b), x0)) * Real.1 +
            pointwise_add(pointwise_mul(constant[Real, Real](c), identity_fn[Real]), constant[Real, Real](b), x0) *
            (
                pointwise_add(pointwise_mul(constant[Real, Real](c), identity_fn[Real]), constant[Real, Real](b), x0) * Real.1 +
                pointwise_add(pointwise_mul(constant[Real, Real](c), identity_fn[Real]), constant[Real, Real](b), x0) * Real.1
            )
        ) * c
    )
} by {
    let affine = pointwise_add(pointwise_mul(constant[Real, Real](c), identity_fn[Real]), constant[Real, Real](b))
    cube_real_has_derivative_at(affine(x0))
    derivative_after_affine(
        c,
        b,
        cube_real,
        x0,
        square_real(affine(x0)) * Real.1 + affine(x0) * (affine(x0) * Real.1 + affine(x0) * Real.1)
    )
    has_derivative_at(
        compose(cube_real, pointwise_add(pointwise_mul(constant[Real, Real](c), identity_fn[Real]), constant[Real, Real](b))),
        x0,
        (square_real(affine(x0)) * Real.1 + affine(x0) * (affine(x0) * Real.1 + affine(x0) * Real.1)) * c
    )
}

/// Composing an affine function after the square function follows the chain rule.
theorem derivative_affine_after_square_real(c: Real, b: Real, x0: Real) {
    has_derivative_at(
        compose(pointwise_add(pointwise_mul(constant[Real, Real](c), identity_fn[Real]), constant[Real, Real](b)), square_real),
        x0,
        c * (x0 * Real.1 + x0 * Real.1)
    )
} by {
    square_real_has_derivative_at(x0)
    derivative_affine_after(c, b, square_real, x0, x0 * Real.1 + x0 * Real.1)
    has_derivative_at(
        compose(pointwise_add(pointwise_mul(constant[Real, Real](c), identity_fn[Real]), constant[Real, Real](b)), square_real),
        x0,
        c * (x0 * Real.1 + x0 * Real.1)
    )
}

/// Composing an affine function after the cube function follows the chain rule.
theorem derivative_affine_after_cube_real(c: Real, b: Real, x0: Real) {
    has_derivative_at(
        compose(pointwise_add(pointwise_mul(constant[Real, Real](c), identity_fn[Real]), constant[Real, Real](b)), cube_real),
        x0,
        c * (square_real(x0) * Real.1 + x0 * (x0 * Real.1 + x0 * Real.1))
    )
} by {
    cube_real_has_derivative_at(x0)
    derivative_affine_after(c, b, cube_real, x0, square_real(x0) * Real.1 + x0 * (x0 * Real.1 + x0 * Real.1))
    has_derivative_at(
        compose(pointwise_add(pointwise_mul(constant[Real, Real](c), identity_fn[Real]), constant[Real, Real](b)), cube_real),
        x0,
        c * (square_real(x0) * Real.1 + x0 * (x0 * Real.1 + x0 * Real.1))
    )
}

/// Differentiability is preserved by composing the square function after an affine function.
theorem differentiable_square_real_after_affine(c: Real, b: Real, x0: Real) {
    differentiable_at(
        compose(square_real, pointwise_add(pointwise_mul(constant[Real, Real](c), identity_fn[Real]), constant[Real, Real](b))),
        x0
    )
} by {
    let affine = pointwise_add(pointwise_mul(constant[Real, Real](c), identity_fn[Real]), constant[Real, Real](b))
    square_real_differentiable_at(affine(x0))
    differentiable_after_affine(c, b, square_real, x0)
    differentiable_at(compose(square_real, affine), x0)
}

/// Differentiability is preserved by composing the cube function after an affine function.
theorem differentiable_cube_real_after_affine(c: Real, b: Real, x0: Real) {
    differentiable_at(
        compose(cube_real, pointwise_add(pointwise_mul(constant[Real, Real](c), identity_fn[Real]), constant[Real, Real](b))),
        x0
    )
} by {
    let affine = pointwise_add(pointwise_mul(constant[Real, Real](c), identity_fn[Real]), constant[Real, Real](b))
    cube_real_differentiable_at(affine(x0))
    differentiable_after_affine(c, b, cube_real, x0)
    differentiable_at(compose(cube_real, affine), x0)
}

/// Differentiability is preserved by composing an affine function after the square function.
theorem differentiable_affine_after_square_real(c: Real, b: Real, x0: Real) {
    differentiable_at(
        compose(pointwise_add(pointwise_mul(constant[Real, Real](c), identity_fn[Real]), constant[Real, Real](b)), square_real),
        x0
    )
} by {
    square_real_differentiable_at(x0)
    differentiable_affine_after(c, b, square_real, x0)
    differentiable_at(compose(pointwise_add(pointwise_mul(constant[Real, Real](c), identity_fn[Real]), constant[Real, Real](b)), square_real), x0)
}

/// Differentiability is preserved by composing an affine function after the cube function.
theorem differentiable_affine_after_cube_real(c: Real, b: Real, x0: Real) {
    differentiable_at(
        compose(pointwise_add(pointwise_mul(constant[Real, Real](c), identity_fn[Real]), constant[Real, Real](b)), cube_real),
        x0
    )
} by {
    cube_real_differentiable_at(x0)
    differentiable_affine_after(c, b, cube_real, x0)
    differentiable_at(compose(pointwise_add(pointwise_mul(constant[Real, Real](c), identity_fn[Real]), constant[Real, Real](b)), cube_real), x0)
}
