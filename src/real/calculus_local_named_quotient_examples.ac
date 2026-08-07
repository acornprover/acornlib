/// Local named rational derivative and differentiability examples.

from data.basic.functions import identity_fn
from real.real_base import Real
from real.continuity_affine import affine_real
from real.continuity_square import square_real
from real.derivative_basic import has_derivative_at, differentiable_at,
    identity_has_derivative_at, identity_differentiable_at
from real.derivative_affine_named import affine_real_has_derivative_at,
    affine_real_differentiable_at
from real.derivative_polynomial_chain import square_real_has_derivative_at,
    square_real_differentiable_at
from real.derivative_quotient import pointwise_div_real, pointwise_reciprocal_real
from real.calculus_quotient_api import reciprocal_has_derivative_at,
    reciprocal_differentiable_at, quotient_has_derivative_at, quotient_differentiable_at

/// The reciprocal of a named affine function has the local reciprocal-rule derivative at nonzero values.
theorem reciprocal_affine_real_has_derivative_at(a: Real, b: Real, x0: Real) {
    affine_real(a, b, x0) != Real.0 implies has_derivative_at(
        pointwise_reciprocal_real(affine_real(a, b)),
        x0,
        (-Real.1 / (affine_real(a, b, x0) * affine_real(a, b, x0))) * a
    )
} by {
    if affine_real(a, b, x0) != Real.0 {
        affine_real_has_derivative_at(a, b, x0)
        reciprocal_has_derivative_at(affine_real(a, b), x0, a)
        has_derivative_at(
            pointwise_reciprocal_real(affine_real(a, b)),
            x0,
            (-Real.1 / (affine_real(a, b, x0) * affine_real(a, b, x0))) * a
        )
    }
}

/// The reciprocal of a named affine function is locally differentiable at nonzero values.
theorem reciprocal_affine_real_differentiable_at(a: Real, b: Real, x0: Real) {
    affine_real(a, b, x0) != Real.0 implies
        differentiable_at(pointwise_reciprocal_real(affine_real(a, b)), x0)
} by {
    if affine_real(a, b, x0) != Real.0 {
        affine_real_differentiable_at(a, b, x0)
        reciprocal_differentiable_at(affine_real(a, b), x0)
        differentiable_at(pointwise_reciprocal_real(affine_real(a, b)), x0)
    }
}

/// The reciprocal of the named square function has the local reciprocal-rule derivative at nonzero values.
theorem reciprocal_square_real_has_derivative_at(x0: Real) {
    square_real(x0) != Real.0 implies has_derivative_at(
        pointwise_reciprocal_real(square_real),
        x0,
        (-Real.1 / (square_real(x0) * square_real(x0))) * (x0 * Real.1 + x0 * Real.1)
    )
} by {
    if square_real(x0) != Real.0 {
        square_real_has_derivative_at(x0)
        reciprocal_has_derivative_at(square_real, x0, x0 * Real.1 + x0 * Real.1)
        has_derivative_at(
            pointwise_reciprocal_real(square_real),
            x0,
            (-Real.1 / (square_real(x0) * square_real(x0))) * (x0 * Real.1 + x0 * Real.1)
        )
    }
}

/// The reciprocal of the named square function is locally differentiable at nonzero values.
theorem reciprocal_square_real_differentiable_at(x0: Real) {
    square_real(x0) != Real.0 implies differentiable_at(pointwise_reciprocal_real(square_real), x0)
} by {
    if square_real(x0) != Real.0 {
        square_real_differentiable_at(x0)
        reciprocal_differentiable_at(square_real, x0)
        differentiable_at(pointwise_reciprocal_real(square_real), x0)
    }
}

/// A named affine numerator over the identity denominator has the local quotient-rule derivative away from zero.
theorem affine_over_identity_has_derivative_at(a: Real, b: Real, x0: Real) {
    identity_fn[Real](x0) != Real.0 implies has_derivative_at(
        pointwise_div_real(affine_real(a, b), identity_fn[Real]),
        x0,
        (a * identity_fn[Real](x0) - affine_real(a, b, x0) * Real.1) /
            (identity_fn[Real](x0) * identity_fn[Real](x0))
    )
} by {
    if identity_fn[Real](x0) != Real.0 {
        affine_real_has_derivative_at(a, b, x0)
        identity_has_derivative_at(x0)
        quotient_has_derivative_at(affine_real(a, b), identity_fn[Real], x0, a, Real.1)
        has_derivative_at(
            pointwise_div_real(affine_real(a, b), identity_fn[Real]),
            x0,
            (a * identity_fn[Real](x0) - affine_real(a, b, x0) * Real.1) /
                (identity_fn[Real](x0) * identity_fn[Real](x0))
        )
    }
}

/// A named affine numerator over the identity denominator is locally differentiable away from zero.
theorem affine_over_identity_differentiable_at(a: Real, b: Real, x0: Real) {
    identity_fn[Real](x0) != Real.0 implies
        differentiable_at(pointwise_div_real(affine_real(a, b), identity_fn[Real]), x0)
} by {
    if identity_fn[Real](x0) != Real.0 {
        affine_real_differentiable_at(a, b, x0)
        identity_differentiable_at(x0)
        quotient_differentiable_at(affine_real(a, b), identity_fn[Real], x0)
        differentiable_at(pointwise_div_real(affine_real(a, b), identity_fn[Real]), x0)
    }
}

/// The named square numerator over the identity denominator has the local quotient-rule derivative away from zero.
theorem square_over_identity_has_derivative_at(x0: Real) {
    identity_fn[Real](x0) != Real.0 implies has_derivative_at(
        pointwise_div_real(square_real, identity_fn[Real]),
        x0,
        ((x0 * Real.1 + x0 * Real.1) * identity_fn[Real](x0) - square_real(x0) * Real.1) /
            (identity_fn[Real](x0) * identity_fn[Real](x0))
    )
} by {
    if identity_fn[Real](x0) != Real.0 {
        square_real_has_derivative_at(x0)
        identity_has_derivative_at(x0)
        quotient_has_derivative_at(square_real, identity_fn[Real], x0, x0 * Real.1 + x0 * Real.1, Real.1)
        has_derivative_at(
            pointwise_div_real(square_real, identity_fn[Real]),
            x0,
            ((x0 * Real.1 + x0 * Real.1) * identity_fn[Real](x0) - square_real(x0) * Real.1) /
                (identity_fn[Real](x0) * identity_fn[Real](x0))
        )
    }
}

/// The named square numerator over the identity denominator is locally differentiable away from zero.
theorem square_over_identity_differentiable_at(x0: Real) {
    identity_fn[Real](x0) != Real.0 implies
        differentiable_at(pointwise_div_real(square_real, identity_fn[Real]), x0)
} by {
    if identity_fn[Real](x0) != Real.0 {
        square_real_differentiable_at(x0)
        identity_differentiable_at(x0)
        quotient_differentiable_at(square_real, identity_fn[Real], x0)
        differentiable_at(pointwise_div_real(square_real, identity_fn[Real]), x0)
    }
}

/// The identity numerator over a named affine denominator has the local quotient-rule derivative at nonzero denominator values.
theorem identity_over_affine_has_derivative_at(a: Real, b: Real, x0: Real) {
    affine_real(a, b, x0) != Real.0 implies has_derivative_at(
        pointwise_div_real(identity_fn[Real], affine_real(a, b)),
        x0,
        (Real.1 * affine_real(a, b, x0) - identity_fn[Real](x0) * a) /
            (affine_real(a, b, x0) * affine_real(a, b, x0))
    )
} by {
    if affine_real(a, b, x0) != Real.0 {
        identity_has_derivative_at(x0)
        affine_real_has_derivative_at(a, b, x0)
        quotient_has_derivative_at(identity_fn[Real], affine_real(a, b), x0, Real.1, a)
        has_derivative_at(
            pointwise_div_real(identity_fn[Real], affine_real(a, b)),
            x0,
            (Real.1 * affine_real(a, b, x0) - identity_fn[Real](x0) * a) /
                (affine_real(a, b, x0) * affine_real(a, b, x0))
        )
    }
}

/// The identity numerator over a named affine denominator is locally differentiable at nonzero denominator values.
theorem identity_over_affine_differentiable_at(a: Real, b: Real, x0: Real) {
    affine_real(a, b, x0) != Real.0 implies
        differentiable_at(pointwise_div_real(identity_fn[Real], affine_real(a, b)), x0)
} by {
    if affine_real(a, b, x0) != Real.0 {
        identity_differentiable_at(x0)
        affine_real_differentiable_at(a, b, x0)
        quotient_differentiable_at(identity_fn[Real], affine_real(a, b), x0)
        differentiable_at(pointwise_div_real(identity_fn[Real], affine_real(a, b)), x0)
    }
}

/// The named square numerator over a named affine denominator has the local quotient-rule derivative at nonzero denominator values.
theorem square_over_affine_has_derivative_at(a: Real, b: Real, x0: Real) {
    affine_real(a, b, x0) != Real.0 implies has_derivative_at(
        pointwise_div_real(square_real, affine_real(a, b)),
        x0,
        ((x0 * Real.1 + x0 * Real.1) * affine_real(a, b, x0) - square_real(x0) * a) /
            (affine_real(a, b, x0) * affine_real(a, b, x0))
    )
} by {
    if affine_real(a, b, x0) != Real.0 {
        square_real_has_derivative_at(x0)
        affine_real_has_derivative_at(a, b, x0)
        quotient_has_derivative_at(square_real, affine_real(a, b), x0, x0 * Real.1 + x0 * Real.1, a)
        has_derivative_at(
            pointwise_div_real(square_real, affine_real(a, b)),
            x0,
            ((x0 * Real.1 + x0 * Real.1) * affine_real(a, b, x0) - square_real(x0) * a) /
                (affine_real(a, b, x0) * affine_real(a, b, x0))
        )
    }
}

/// The named square numerator over a named affine denominator is locally differentiable at nonzero denominator values.
theorem square_over_affine_differentiable_at(a: Real, b: Real, x0: Real) {
    affine_real(a, b, x0) != Real.0 implies
        differentiable_at(pointwise_div_real(square_real, affine_real(a, b)), x0)
} by {
    if affine_real(a, b, x0) != Real.0 {
        square_real_differentiable_at(x0)
        affine_real_differentiable_at(a, b, x0)
        quotient_differentiable_at(square_real, affine_real(a, b), x0)
        differentiable_at(pointwise_div_real(square_real, affine_real(a, b)), x0)
    }
}
