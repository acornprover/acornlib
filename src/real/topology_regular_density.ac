from data.basic.set import Set, all_sets_subset_universal, compl_of_compl_is_self, double_inclusion,
    empty_set_compl_is_universal, empty_set_is_always_subset
from real.real_field import Real
from real.topology import closure, interior
from real.topology_closure_interior_duality import interior_eq_closure_complement_complement
from real.topology_codense import interior_of_codense_real_set_is_empty, is_codense_real_set
from real.topology_dense import is_dense_real_set
from real.topology_interior_closure_idempotent import closure_empty_real_set,
    interior_universal_real_set
from real.topology_nowhere_dense import is_nowhere_dense_real_set
from real.topology_regular_sets import interior_subset_regular_closed_hull,
    is_regular_closed_real_set, is_regular_open_real_set, regular_closed_hull,
    regular_open_hull, regular_open_hull_subset_closure

/// A real set with empty interior is codense.
theorem codense_real_set_of_empty_interior(s: Set[Real]) {
    interior(s) = Set[Real].empty_set implies is_codense_real_set(s)
} by {
    if interior(s) = Set[Real].empty_set {
        interior_eq_closure_complement_complement(s)
        interior(s) = closure(s.c).c
        closure(s.c).c = Set[Real].empty_set
        closure(s.c).c.c = Set[Real].empty_set.c
        compl_of_compl_is_self[Real](closure(s.c))
        closure(s.c).c.c = closure(s.c)
        empty_set_compl_is_universal[Real]
        Set[Real].empty_set.c = Set[Real].universal_set
        closure(s.c) = Set[Real].universal_set
        is_dense_real_set(s.c)
        is_codense_real_set(s)
    }
}

/// Codensity is equivalent to empty interior.
theorem codense_real_set_eq_empty_interior(s: Set[Real]) {
    is_codense_real_set(s) = (interior(s) = Set[Real].empty_set)
} by {
    if is_codense_real_set(s) {
        interior_of_codense_real_set_is_empty(s)
        interior(s) = Set[Real].empty_set
    }
    if interior(s) = Set[Real].empty_set {
        codense_real_set_of_empty_interior(s)
        is_codense_real_set(s)
    }
    is_codense_real_set(s) = (interior(s) = Set[Real].empty_set)
}

/// A dense real set has universal regular-open hull.
theorem dense_real_set_imp_regular_open_hull_universal(s: Set[Real]) {
    is_dense_real_set(s) implies regular_open_hull(s) = Set[Real].universal_set
} by {
    if is_dense_real_set(s) {
        is_dense_real_set(s) = (closure(s) = Set[Real].universal_set)
        closure(s) = Set[Real].universal_set
        interior_universal_real_set
        interior(Set[Real].universal_set) = Set[Real].universal_set
        regular_open_hull(s) = interior(closure(s))
        regular_open_hull(s) = Set[Real].universal_set
    }
}

/// A real set with universal regular-open hull is dense.
theorem regular_open_hull_universal_imp_dense_real_set(s: Set[Real]) {
    regular_open_hull(s) = Set[Real].universal_set implies is_dense_real_set(s)
} by {
    if regular_open_hull(s) = Set[Real].universal_set {
        regular_open_hull_subset_closure(s)
        regular_open_hull(s).subset(closure(s))
        Set[Real].universal_set.subset(closure(s))
        all_sets_subset_universal[Real](closure(s))
        closure(s).subset(Set[Real].universal_set)
        double_inclusion(closure(s), Set[Real].universal_set)
        closure(s) = Set[Real].universal_set
        is_dense_real_set(s)
    }
}

/// Density is equivalent to universal regular-open hull.
theorem regular_open_hull_universal_eq_dense_real_set(s: Set[Real]) {
    (regular_open_hull(s) = Set[Real].universal_set) = is_dense_real_set(s)
} by {
    if regular_open_hull(s) = Set[Real].universal_set {
        regular_open_hull_universal_imp_dense_real_set(s)
        is_dense_real_set(s)
    }
    if is_dense_real_set(s) {
        dense_real_set_imp_regular_open_hull_universal(s)
        regular_open_hull(s) = Set[Real].universal_set
    }
    (regular_open_hull(s) = Set[Real].universal_set) = is_dense_real_set(s)
}

/// A nowhere dense real set has empty regular-open hull.
theorem nowhere_dense_real_set_imp_regular_open_hull_empty(s: Set[Real]) {
    is_nowhere_dense_real_set(s) implies regular_open_hull(s) = Set[Real].empty_set
} by {
    if is_nowhere_dense_real_set(s) {
        is_nowhere_dense_real_set(s) = (interior(closure(s)) = Set[Real].empty_set)
        interior(closure(s)) = Set[Real].empty_set
        regular_open_hull(s) = interior(closure(s))
        regular_open_hull(s) = Set[Real].empty_set
    }
}

/// A real set with empty regular-open hull is nowhere dense.
theorem regular_open_hull_empty_imp_nowhere_dense_real_set(s: Set[Real]) {
    regular_open_hull(s) = Set[Real].empty_set implies is_nowhere_dense_real_set(s)
} by {
    if regular_open_hull(s) = Set[Real].empty_set {
        regular_open_hull(s) = interior(closure(s))
        interior(closure(s)) = Set[Real].empty_set
        is_nowhere_dense_real_set(s)
    }
}

/// Nowhere density is equivalent to empty regular-open hull.
theorem regular_open_hull_empty_eq_nowhere_dense_real_set(s: Set[Real]) {
    (regular_open_hull(s) = Set[Real].empty_set) = is_nowhere_dense_real_set(s)
} by {
    if regular_open_hull(s) = Set[Real].empty_set {
        regular_open_hull_empty_imp_nowhere_dense_real_set(s)
        is_nowhere_dense_real_set(s)
    }
    if is_nowhere_dense_real_set(s) {
        nowhere_dense_real_set_imp_regular_open_hull_empty(s)
        regular_open_hull(s) = Set[Real].empty_set
    }
    (regular_open_hull(s) = Set[Real].empty_set) = is_nowhere_dense_real_set(s)
}

/// A codense real set has empty regular-closed hull.
theorem codense_real_set_imp_regular_closed_hull_empty(s: Set[Real]) {
    is_codense_real_set(s) implies regular_closed_hull(s) = Set[Real].empty_set
} by {
    if is_codense_real_set(s) {
        interior_of_codense_real_set_is_empty(s)
        interior(s) = Set[Real].empty_set
        closure_empty_real_set
        closure(Set[Real].empty_set) = Set[Real].empty_set
        regular_closed_hull(s) = closure(interior(s))
        regular_closed_hull(s) = Set[Real].empty_set
    }
}

/// A real set with empty regular-closed hull is codense.
theorem regular_closed_hull_empty_imp_codense_real_set(s: Set[Real]) {
    regular_closed_hull(s) = Set[Real].empty_set implies is_codense_real_set(s)
} by {
    if regular_closed_hull(s) = Set[Real].empty_set {
        interior_subset_regular_closed_hull(s)
        interior(s).subset(regular_closed_hull(s))
        interior(s).subset(Set[Real].empty_set)
        empty_set_is_always_subset[Real](interior(s))
        Set[Real].empty_set.subset(interior(s))
        double_inclusion(interior(s), Set[Real].empty_set)
        interior(s) = Set[Real].empty_set
        codense_real_set_of_empty_interior(s)
        is_codense_real_set(s)
    }
}

/// Codensity is equivalent to empty regular-closed hull.
theorem regular_closed_hull_empty_eq_codense_real_set(s: Set[Real]) {
    (regular_closed_hull(s) = Set[Real].empty_set) = is_codense_real_set(s)
} by {
    if regular_closed_hull(s) = Set[Real].empty_set {
        regular_closed_hull_empty_imp_codense_real_set(s)
        is_codense_real_set(s)
    }
    if is_codense_real_set(s) {
        codense_real_set_imp_regular_closed_hull_empty(s)
        regular_closed_hull(s) = Set[Real].empty_set
    }
    (regular_closed_hull(s) = Set[Real].empty_set) = is_codense_real_set(s)
}

/// The regular-closed hull is universal exactly when the interior is dense.
theorem regular_closed_hull_universal_eq_dense_interior(s: Set[Real]) {
    (regular_closed_hull(s) = Set[Real].universal_set) = is_dense_real_set(interior(s))
} by {
    if regular_closed_hull(s) = Set[Real].universal_set {
        regular_closed_hull(s) = closure(interior(s))
        closure(interior(s)) = Set[Real].universal_set
        is_dense_real_set(interior(s))
    }
    if is_dense_real_set(interior(s)) {
        is_dense_real_set(interior(s)) = (closure(interior(s)) = Set[Real].universal_set)
        closure(interior(s)) = Set[Real].universal_set
        regular_closed_hull(s) = closure(interior(s))
        regular_closed_hull(s) = Set[Real].universal_set
    }
    (regular_closed_hull(s) = Set[Real].universal_set) = is_dense_real_set(interior(s))
}

/// A dense regular-open real set is universal.
theorem dense_regular_open_real_set_is_universal(s: Set[Real]) {
    is_dense_real_set(s) and is_regular_open_real_set(s) implies s = Set[Real].universal_set
} by {
    if is_dense_real_set(s) and is_regular_open_real_set(s) {
        dense_real_set_imp_regular_open_hull_universal(s)
        regular_open_hull(s) = Set[Real].universal_set
        is_regular_open_real_set(s) = (s = regular_open_hull(s))
        s = regular_open_hull(s)
        s = Set[Real].universal_set
    }
}

/// A nowhere dense regular-open real set is empty.
theorem nowhere_dense_regular_open_real_set_is_empty(s: Set[Real]) {
    is_nowhere_dense_real_set(s) and is_regular_open_real_set(s) implies s = Set[Real].empty_set
} by {
    if is_nowhere_dense_real_set(s) and is_regular_open_real_set(s) {
        nowhere_dense_real_set_imp_regular_open_hull_empty(s)
        regular_open_hull(s) = Set[Real].empty_set
        is_regular_open_real_set(s) = (s = regular_open_hull(s))
        s = regular_open_hull(s)
        s = Set[Real].empty_set
    }
}

/// A codense regular-closed real set is empty.
theorem codense_regular_closed_real_set_is_empty(s: Set[Real]) {
    is_codense_real_set(s) and is_regular_closed_real_set(s) implies s = Set[Real].empty_set
} by {
    if is_codense_real_set(s) and is_regular_closed_real_set(s) {
        codense_real_set_imp_regular_closed_hull_empty(s)
        regular_closed_hull(s) = Set[Real].empty_set
        is_regular_closed_real_set(s) = (s = regular_closed_hull(s))
        s = regular_closed_hull(s)
        s = Set[Real].empty_set
    }
}
