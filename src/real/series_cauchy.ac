/// The Cauchy criterion for real-valued infinite series.
from nat import Nat, lte_trans
from list import map, sum, partial
from real.real_base import Real, close_comm
from real.real_seq import cauchy_bound, converges
from real.real_series import diff_partial
from real.cauchy_criterion import is_cauchy_seq, is_cauchy_seq_at, cauchy_imp_converges, converges_imp_cauchy_seq
from real.abs_conv import abs_fn, absolutely_converges_imp_converges

/// True if the partial sums of `a` form a Cauchy sequence in the windowed
/// form: for every positive epsilon, every sufficiently late finite block
/// `a(k) + ... + a(m - 1)` has absolute value less than epsilon.
define is_cauchy_series(a: Nat -> Real) -> Bool {
    forall(eps: Real) {
        eps.is_positive implies exists(n: Nat) {
            forall(k: Nat, m: Nat) {
                n <= k and k <= m implies sum(map(k.until(m), a)).abs < eps
            }
        }
    }
}

/// A windowed-Cauchy series has Cauchy partial sums.
theorem cauchy_series_imp_partial_cauchy(a: Nat -> Real) {
    is_cauchy_series(a) implies is_cauchy_seq(partial(a))
} by {
    let p = partial(a)
    if is_cauchy_series(a) {
        is_cauchy_series(a) = forall(e: Real) {
            e.is_positive implies exists(n: Nat) {
                forall(k: Nat, m: Nat) {
                    n <= k and k <= m implies sum(map(k.until(m), a)).abs < e
                }
            }
        }
        forall(eps: Real) {
            if eps.is_positive {
                exists(n: Nat) {
                    forall(k: Nat, m: Nat) {
                        n <= k and k <= m implies sum(map(k.until(m), a)).abs < eps
                    }
                }
                let n: Nat satisfy {
                    forall(k: Nat, m: Nat) {
                        n <= k and k <= m implies sum(map(k.until(m), a)).abs < eps
                    }
                }
                forall(i: Nat, j: Nat) {
                    if n <= i and n <= j {
                        if i <= j {
                            partial(a, j) - partial(a, i) = sum(map(i.until(j), a))
                            sum(map(i.until(j), a)).abs < eps
                            (p(j) - p(i)).abs < eps
                            p(j).is_close(p(i), eps)
                            close_comm(p(j), p(i), eps)
                            p(i).is_close(p(j), eps)
                        } else {
                            j <= i
                            partial(a, i) - partial(a, j) = sum(map(j.until(i), a))
                            sum(map(j.until(i), a)).abs < eps
                            (p(i) - p(j)).abs < eps
                            p(i).is_close(p(j), eps)
                        }
                    }
                }
                exists(k0: Nat, k1: Nat) {
                    n <= k0 and n <= k1 and not p(k0).is_close(p(k1), eps)
                } or cauchy_bound(p, n, eps)
                cauchy_bound(p, n, eps)
                exists(n0: Nat) {
                    cauchy_bound(p, n0, eps)
                }
            }
        }
    }
}

/// Partial sums that are Cauchy give a windowed-Cauchy series.
theorem partial_cauchy_imp_cauchy_series(a: Nat -> Real) {
    is_cauchy_seq(partial(a)) implies is_cauchy_series(a)
} by {
    let p = partial(a)
    if is_cauchy_seq(p) {
        is_cauchy_seq(p) = forall(e: Real) {
            is_cauchy_seq_at(p, e)
        }
        forall(eps: Real) {
            if eps.is_positive {
                is_cauchy_seq_at(p, eps)
                is_cauchy_seq_at(p, eps) = (eps.is_positive implies exists(n: Nat) {
                    forall(i: Nat, j: Nat) {
                        n <= i and n <= j implies p(i).is_close(p(j), eps)
                    }
                })
                eps.is_positive implies exists(n: Nat) {
                    forall(i: Nat, j: Nat) {
                        n <= i and n <= j implies p(i).is_close(p(j), eps)
                    }
                }
                exists(n: Nat) {
                    forall(i: Nat, j: Nat) {
                        n <= i and n <= j implies p(i).is_close(p(j), eps)
                    }
                }
                let n: Nat satisfy {
                    forall(i: Nat, j: Nat) {
                        n <= i and n <= j implies p(i).is_close(p(j), eps)
                    }
                }
                forall(k: Nat, m: Nat) {
                    if n <= k and k <= m {
                        n <= m
                        p(k).is_close(p(m), eps)
                        close_comm(p(k), p(m), eps)
                        p(m).is_close(p(k), eps)
                        (p(m) - p(k)).abs < eps
                        partial(a, m) - partial(a, k) = sum(map(k.until(m), a))
                        sum(map(k.until(m), a)).abs < eps
                    }
                }
                exists(k0: Nat, k1: Nat) {
                    n <= k0 and k0 <= k1 and not sum(map(k0.until(k1), a)).abs < eps
                } or forall(k: Nat, m: Nat) {
                    n <= k and k <= m implies sum(map(k.until(m), a)).abs < eps
                }
                forall(k: Nat, m: Nat) {
                    n <= k and k <= m implies sum(map(k.until(m), a)).abs < eps
                }
                exists(n0: Nat) {
                    forall(k: Nat, m: Nat) {
                        n0 <= k and k <= m implies sum(map(k.until(m), a)).abs < eps
                    }
                }
            }
        }
    }
}

/// Cauchy criterion for series (forward): a series whose partial sums
/// converge satisfies the windowed Cauchy condition.
theorem converges_imp_cauchy_series(a: Nat -> Real) {
    converges(partial(a)) implies is_cauchy_series(a)
} by {
    if converges(partial(a)) {
        converges_imp_cauchy_seq(partial(a))
        partial_cauchy_imp_cauchy_series(a)
    }
}

/// Cauchy criterion for series (backward): if the windowed Cauchy condition
/// holds, the partial sums converge.
theorem cauchy_series_imp_converges(a: Nat -> Real) {
    is_cauchy_series(a) implies converges(partial(a))
} by {
    if is_cauchy_series(a) {
        cauchy_series_imp_partial_cauchy(a)
        cauchy_imp_converges(partial(a))
    }
}

/// If the series of absolute values is windowed-Cauchy, then so is the
/// original series. Cauchy-criterion form of "absolute convergence implies
/// convergence".
theorem abs_cauchy_series_imp_cauchy_series(a: Nat -> Real) {
    is_cauchy_series(abs_fn(a)) implies is_cauchy_series(a)
} by {
    if is_cauchy_series(abs_fn(a)) {
        cauchy_series_imp_converges(abs_fn(a))
        // converges(partial(abs_fn(a))) is the definition of absolute convergence
        absolutely_converges_imp_converges(a)
        converges_imp_cauchy_series(a)
    }
}
