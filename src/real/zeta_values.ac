// The values of the Riemann zeta function at positive even integers, starting
// with zeta(2).  The zeta(2) series is the sum of the squared reciprocals,
// sum_{n=1}^{infinity} 1 / n^2, whose partial sums are indexed from n = 0 by
// the terms 1 / (n + 1)^2 (see src/theorems1000/theorem_basel.ac for the
// classical statement).  This file proves the crude bound on the partial
// sums (they are at most two, via comparison with the telescoping series
// sum 2 / ((n + 1)(n + 2)) = 2) and the convergence of the series, and it
// records the alternating (Mercator) series for ln 2 together with the
// classical values zeta(2) = pi^2 / 6 and zeta(4) = pi^4 / 90, which the
// library does not yet prove.

from nat import Nat, from_nat, from_nat_add, from_nat_mul, from_nat_one, add_suc_right, add_one_right
from list import partial, partial_pointwise_eq
from real.real_base import Real, lte_add_right, lte_trans, neg_neg
from real.harmonic import harmonic, harmonic_nonnegative, from_nat_suc_pos_real, real_one_div_pos, real_recip_antitone_pos, nat_recip_antitone_suc, from_nat_real_suc_lte_of_nat_lte
from real.real_field import div_mul_cancel_right, div_mul_cancel_left, div_cancel_common, mul_le_mul_pos_right, mul_le_mul_pos_left
from algebra.field.field import mul_not_zero
from real.real_ring import converges, converges_to, limit, mul_pos_pos
from real.real_seq import converges_imp_converges_to
from real.real_series import seq_lte, partial_seq_lte, partial_mul_seq_comm, mul_seq, is_lower_bound, is_upper_bound, is_increasing, nonneg_partial_increasing, monotone_convergence_principle, partial_suc, partial_zero
from real.integral import diff_step, telescope, neg_lte_flip
from real.trig_identities import div_sub_same_denom, div_add_same_denom
from real.series_tests import alt_term, is_decreasing_seq
from real.exp import two_positive
from algebra.ring.ring import alternating_sign, alternating_sign_zero, alternating_sign_suc

numerals Real
numerals Nat

/// The kth term of the zeta(2) series: 1 / (k + 1)^2, indexed from zero.
define zeta2_term(k: Nat) -> Real {
    Real.1 / (from_nat[Real](k.suc) * from_nat[Real](k.suc))
}

/// The kth telescoping majorant term: 1 / (k + 1) - 1 / (k + 2).
define zeta2_tel_term(k: Nat) -> Real {
    harmonic(k) - harmonic(k.suc)
}

/// The negated harmonic terms, whose successive differences telescope.
define zeta2_neg_harmonic(k: Nat) -> Real {
    -harmonic(k)
}

/// The difference of two unit fractions is the difference of the denominators
/// over the product of the denominators.
theorem recip_diff(a: Real, b: Real) {
    a != Real.0 and b != Real.0
    implies Real.1 / a - Real.1 / b = (b - a) / (a * b)
} by {
    if a != Real.0 and b != Real.0 {
        div_mul_cancel_right(b, a)
        b / (a * b) = Real.1 / a
        Real.1 / a = b / (a * b)
        div_mul_cancel_right(a, b)
        a / (b * a) = Real.1 / b
        b * a = a * b
        a / (a * b) = Real.1 / b
        Real.1 / b = a / (a * b)
        mul_not_zero[Real](a, b)
        a * b != Real.0
        Real.1 / a - Real.1 / b = b / (a * b) - a / (a * b)
        div_sub_same_denom(Real.1, b, Real.1, a, a * b)
        Real.1 * b / (a * b) - Real.1 * a / (a * b) = (Real.1 * b - Real.1 * a) / (a * b)
        Real.1 * b / (a * b) = b / (a * b)
        Real.1 * a / (a * b) = a / (a * b)
        b / (a * b) - a / (a * b) = (b - a) / (a * b)
        Real.1 / a - Real.1 / b = (b - a) / (a * b)
    }
}

/// A squared reciprocal is at most twice the telescoping difference of the
/// unit fractions, provided the first denominator is at least one and the
/// second exceeds the first by one.
theorem recip_sq_le_twice_tel_diff(a: Real, b: Real) {
    a > Real.0 and b > Real.0 and b = a + Real.1 and Real.1 <= a
    implies Real.1 / (a * a) <= (Real.1 + Real.1) * (Real.1 / a - Real.1 / b)
} by {
    if a > Real.0 and b > Real.0 and b = a + Real.1 and Real.1 <= a {
        a != Real.0
        b != Real.0
        // The first denominator is at most the second.
        Real.1.is_positive
        Real.0 <= Real.1
        lte_add_right(Real.0, Real.1, a)
        Real.0 + a <= Real.1 + a
        Real.0 + a = a
        Real.1 + a = a + Real.1
        a <= a + Real.1
        b = a + Real.1
        a <= b
        real_recip_antitone_pos(a, b)
        Real.1 / b <= Real.1 / a
        // The first denominator is at most its square: a <= a * a.
        mul_le_mul_pos_right(Real.1, a, a)
        Real.1 * a <= a * a
        Real.1 * a = a
        a <= a * a
        // The second denominator is at most twice the first: b <= a + a.
        lte_add_right(Real.1, a, a)
        Real.1 + a <= a + a
        b = a + Real.1
        b <= a + a
        // The product is at most twice the square: a * b <= 2 * (a * a).
        mul_le_mul_pos_right(b, a + a, a)
        b * a <= (a + a) * a
        b * a = a * b
        (a + a) * a = (Real.1 + Real.1) * (a * a)
        a * b <= (Real.1 + Real.1) * (a * a)
        // Reciprocals reverse the ordering of the positive products.
        a.is_positive
        b.is_positive
        a.is_positive and b.is_positive
        mul_pos_pos(a, b)
        (a * b).is_positive
        (Real.1 + Real.1).is_positive
        (Real.1 + Real.1).is_positive and (a * a).is_positive
        mul_pos_pos(Real.1 + Real.1, a * a)
        ((Real.1 + Real.1) * (a * a)).is_positive
        (a * b) > Real.0
        a * b <= (Real.1 + Real.1) * (a * a)
        real_recip_antitone_pos(a * b, (Real.1 + Real.1) * (a * a))
        Real.1 / ((Real.1 + Real.1) * (a * a)) <= Real.1 / (a * b)
        // Multiply by two on both sides.
        (Real.1 + Real.1) > Real.0
        Real.1 / ((Real.1 + Real.1) * (a * a)) <= Real.1 / (a * b) and (Real.1 + Real.1) > Real.0
        mul_le_mul_pos_left(Real.1 / ((Real.1 + Real.1) * (a * a)), Real.1 / (a * b), Real.1 + Real.1)
        (Real.1 + Real.1) * (Real.1 / ((Real.1 + Real.1) * (a * a))) <= (Real.1 + Real.1) * (Real.1 / (a * b))
        // The left-hand side is 1 / (a * a): 2 * (1 / (2 * (a * a))) = 2 / (2 * (a * a)) = 1 / (a * a).
        (Real.1 + Real.1) * (Real.1 / ((Real.1 + Real.1) * (a * a))) = (Real.1 + Real.1) / ((Real.1 + Real.1) * (a * a))
        (Real.1 + Real.1) * (a * a) = (a * a) * (Real.1 + Real.1)
        (Real.1 + Real.1) / ((Real.1 + Real.1) * (a * a)) = (Real.1 + Real.1) / ((a * a) * (Real.1 + Real.1))
        (Real.1 + Real.1).is_positive
        (Real.1 + Real.1) != Real.0
        mul_not_zero[Real](a, a)
        a * a != Real.0
        div_mul_cancel_right(Real.1 + Real.1, a * a)
        (Real.1 + Real.1) / ((a * a) * (Real.1 + Real.1)) = Real.1 / (a * a)
        (Real.1 + Real.1) * (Real.1 / ((Real.1 + Real.1) * (a * a))) = Real.1 / (a * a)
        // The difference of unit fractions is the reciprocal of the product.
        recip_diff(a, b)
        Real.1 / a - Real.1 / b = (b - a) / (a * b)
        b - a = Real.1
        (b - a) / (a * b) = Real.1 / (a * b)
        Real.1 / (a * b) = Real.1 / a - Real.1 / b
        (Real.1 + Real.1) * (Real.1 / (a * b)) = (Real.1 + Real.1) * (Real.1 / a - Real.1 / b)
        Real.1 / (a * a) <= (Real.1 + Real.1) * (Real.1 / a - Real.1 / b)
    }
}

/// The squared-reciprocal terms are positive.
theorem zeta2_term_pos(k: Nat) {
    zeta2_term(k) > Real.0
} by {
    from_nat_suc_pos_real(k)
    from_nat[Real](k.suc) > Real.0
    mul_pos_pos(from_nat[Real](k.suc), from_nat[Real](k.suc))
    (from_nat[Real](k.suc) * from_nat[Real](k.suc)).is_positive
    real_one_div_pos(from_nat[Real](k.suc) * from_nat[Real](k.suc))
    Real.1 / (from_nat[Real](k.suc) * from_nat[Real](k.suc)) > Real.0
    zeta2_term(k) = Real.1 / (from_nat[Real](k.suc) * from_nat[Real](k.suc))
    zeta2_term(k) > Real.0
}

/// The squared-reciprocal terms are nonnegative.
theorem zeta2_term_nonneg(k: Nat) {
    Real.0 <= zeta2_term(k)
} by {
    zeta2_term_pos(k)
    zeta2_term(k) > Real.0
    Real.0 <= zeta2_term(k)
}

/// Each squared-reciprocal term is at most twice the telescoping majorant.
theorem zeta2_term_le_majorant(k: Nat) {
    zeta2_term(k) <= (Real.1 + Real.1) * zeta2_tel_term(k)
} by {
    from_nat_suc_pos_real(k)
    from_nat[Real](k.suc) > Real.0
    from_nat_suc_pos_real(k.suc)
    from_nat[Real](k.suc.suc) > Real.0
    from_nat_add[Real](k.suc, Nat.1)
    from_nat[Real](k.suc + Nat.1) = from_nat[Real](k.suc) + from_nat[Real](Nat.1)
    from_nat_one[Real]
    from_nat[Real](Nat.1) = Real.1
    k.suc + Nat.1 = k.suc.suc
    from_nat[Real](k.suc.suc) = from_nat[Real](k.suc) + Real.1
    // The first denominator is at least one.
    Nat.0 <= k
    from_nat_real_suc_lte_of_nat_lte(Nat.0, k)
    from_nat[Real](Nat.1) <= from_nat[Real](k.suc)
    from_nat_one[Real]
    from_nat[Real](Nat.1) = Real.1
    Real.1 <= from_nat[Real](k.suc)
    recip_sq_le_twice_tel_diff(from_nat[Real](k.suc), from_nat[Real](k.suc.suc))
    Real.1 / (from_nat[Real](k.suc) * from_nat[Real](k.suc)) <= (Real.1 + Real.1) * (Real.1 / from_nat[Real](k.suc) - Real.1 / from_nat[Real](k.suc.suc))
    zeta2_term(k) = Real.1 / (from_nat[Real](k.suc) * from_nat[Real](k.suc))
    zeta2_tel_term(k) = harmonic(k) - harmonic(k.suc)
    harmonic(k) = Real.1 / from_nat[Real](k.suc)
    harmonic(k.suc) = Real.1 / from_nat[Real](k.suc.suc)
    zeta2_tel_term(k) = Real.1 / from_nat[Real](k.suc) - Real.1 / from_nat[Real](k.suc.suc)
    zeta2_term(k) <= (Real.1 + Real.1) * zeta2_tel_term(k)
}

/// The telescoping majorant is the successive difference of the negated
/// harmonic terms.
theorem zeta2_tel_diff(k: Nat) {
    diff_step(zeta2_neg_harmonic, k) = zeta2_tel_term(k)
} by {
    diff_step(zeta2_neg_harmonic, k) = zeta2_neg_harmonic(k + Nat.1) - zeta2_neg_harmonic(k)
    zeta2_neg_harmonic(k + Nat.1) = -harmonic(k + Nat.1)
    zeta2_neg_harmonic(k) = -harmonic(k)
    k + Nat.1 = k.suc
    harmonic(k + Nat.1) = harmonic(k.suc)
    neg_neg(harmonic(k))
    -(-harmonic(k)) = harmonic(k)
    -harmonic(k.suc) - (-harmonic(k)) = -harmonic(k.suc) + -(-harmonic(k))
    -harmonic(k.suc) + -(-harmonic(k)) = -harmonic(k.suc) + harmonic(k)
    -harmonic(k.suc) + harmonic(k) = harmonic(k) - harmonic(k.suc)
    -harmonic(k + Nat.1) - (-harmonic(k)) = harmonic(k) - harmonic(k + Nat.1)
    diff_step(zeta2_neg_harmonic, k) = harmonic(k) - harmonic(k.suc)
    zeta2_tel_term(k) = harmonic(k) - harmonic(k.suc)
    diff_step(zeta2_neg_harmonic, k) = zeta2_tel_term(k)
}

/// The telescoping majorant sums to one minus the next harmonic term.
theorem zeta2_tel_partial(n: Nat) {
    partial(zeta2_tel_term, n) = Real.1 - harmonic(n)
} by {
    forall(k: Nat) {
        zeta2_tel_diff(k)
        diff_step(zeta2_neg_harmonic, k) = zeta2_tel_term(k)
    }
    diff_step(zeta2_neg_harmonic) = zeta2_tel_term
    telescope(zeta2_neg_harmonic, n)
    partial(diff_step(zeta2_neg_harmonic), n) = zeta2_neg_harmonic(n) - zeta2_neg_harmonic(Nat.0)
    partial(zeta2_tel_term, n) = zeta2_neg_harmonic(n) - zeta2_neg_harmonic(Nat.0)
    zeta2_neg_harmonic(n) = -harmonic(n)
    zeta2_neg_harmonic(Nat.0) = -harmonic(Nat.0)
    harmonic(Nat.0) = Real.1
    neg_neg(harmonic(Nat.0))
    -(-harmonic(Nat.0)) = harmonic(Nat.0)
    -harmonic(n) - (-harmonic(Nat.0)) = -harmonic(n) + -(-harmonic(Nat.0))
    -harmonic(n) + -(-harmonic(Nat.0)) = -harmonic(n) + harmonic(Nat.0)
    -harmonic(n) + harmonic(Nat.0) = Real.1 - harmonic(n)
    partial(zeta2_tel_term, n) = Real.1 - harmonic(n)
}

/// The partial sums of the telescoping majorant are at most one.
theorem zeta2_tel_partial_le_one(n: Nat) {
    partial(zeta2_tel_term, n) <= Real.1
} by {
    zeta2_tel_partial(n)
    partial(zeta2_tel_term, n) = Real.1 - harmonic(n)
    harmonic_nonnegative(n)
    Real.0 <= harmonic(n)
    neg_lte_flip(Real.0, harmonic(n))
    -harmonic(n) <= -Real.0
    -Real.0 = Real.0
    -harmonic(n) <= Real.0
    lte_add_right(-harmonic(n), Real.0, Real.1)
    -harmonic(n) + Real.1 <= Real.0 + Real.1
    Real.0 + Real.1 = Real.1
    Real.1 - harmonic(n) <= Real.1
    partial(zeta2_tel_term, n) <= Real.1
}

/// The partial sums of the zeta(2) series are at most two.
theorem zeta2_partial_le_two(n: Nat) {
    partial(zeta2_term, n) <= (Real.1 + Real.1)
} by {
    forall(k: Nat) {
        zeta2_term_le_majorant(k)
        zeta2_term(k) <= (Real.1 + Real.1) * zeta2_tel_term(k)
    }
    seq_lte(zeta2_term, mul_seq(Real.1 + Real.1, zeta2_tel_term))
    partial_seq_lte(zeta2_term, mul_seq(Real.1 + Real.1, zeta2_tel_term))
    seq_lte(partial(zeta2_term), partial(mul_seq(Real.1 + Real.1, zeta2_tel_term)))
    partial(zeta2_term, n) <= partial(mul_seq(Real.1 + Real.1, zeta2_tel_term), n)
    partial_mul_seq_comm(Real.1 + Real.1, zeta2_tel_term)
    partial(mul_seq(Real.1 + Real.1, zeta2_tel_term)) = mul_seq(Real.1 + Real.1, partial(zeta2_tel_term))
    partial(mul_seq(Real.1 + Real.1, zeta2_tel_term), n) =
        mul_seq(Real.1 + Real.1, partial(zeta2_tel_term), n)
    mul_seq(Real.1 + Real.1, partial(zeta2_tel_term), n) =
        (Real.1 + Real.1) * partial(zeta2_tel_term, n)
    partial(zeta2_term, n) <= (Real.1 + Real.1) * partial(zeta2_tel_term, n)
    zeta2_tel_partial_le_one(n)
    partial(zeta2_tel_term, n) <= Real.1
    two_positive
    (Real.1 + Real.1).is_positive
    mul_le_mul_pos_left(partial(zeta2_tel_term, n), Real.1, Real.1 + Real.1)
    (Real.1 + Real.1) * partial(zeta2_tel_term, n) <= (Real.1 + Real.1) * Real.1
    (Real.1 + Real.1) * Real.1 = Real.1 + Real.1
    lte_trans(partial(zeta2_term, n), (Real.1 + Real.1) * partial(zeta2_tel_term, n), Real.1 + Real.1)
    partial(zeta2_term, n) <= Real.1 + Real.1
}

/// The zeta(2) series converges.
theorem zeta2_converges {
    converges(partial(zeta2_term))
} by {
    forall(k: Nat) {
        zeta2_term_nonneg(k)
        Real.0 <= zeta2_term(k)
    }
    is_lower_bound(zeta2_term, Real.0)
    nonneg_partial_increasing(zeta2_term)
    is_increasing(partial(zeta2_term))
    forall(n: Nat) {
        zeta2_partial_le_two(n)
        partial(zeta2_term, n) <= Real.1 + Real.1
    }
    is_upper_bound(partial(zeta2_term), Real.1 + Real.1)
    monotone_convergence_principle(partial(zeta2_term), Real.1 + Real.1)
    converges(partial(zeta2_term))
}

/// The zeta(2) series converges to its limit: the partial sums of the squared
/// reciprocals form a convergent sequence.
theorem zeta2_converges_to_limit {
    converges_to(partial(zeta2_term), limit(partial(zeta2_term)))
} by {
    zeta2_converges
    converges(partial(zeta2_term))
    converges_imp_converges_to(partial(zeta2_term))
    converges_to(partial(zeta2_term), limit(partial(zeta2_term)))
}

// ---------------------------------------------------------------------------
// The value of zeta(2): the Basel problem.
//
// The classical statement is
//
//     converges_to(partial(function(n: Nat) {
//         Real.1 / ((from_nat[Real](n) + Real.1) * (from_nat[Real](n) + Real.1))
//     }, k), pi * pi / from_nat[Real](Nat.6))
//
// i.e. sum_{n=1}^{infinity} 1 / n^2 = pi^2 / 6.  It is recorded (as a
// comment, unproved) in src/theorems1000/theorem_basel.ac.  A proof needs the
// Weierstrass sine product or the Fourier series of x^2 on [-pi, pi]; the
// library's Wallis product (src/real/wallis_product.ac) develops a similar
// infinite product but the main limit is not yet closed, so the value is
// stated here for reference and left commented out.
//
// theorem basel_zeta_two {
//     converges_to(
//         function(k: Nat) {
//             partial(function(n: Nat) {
//                 Real.1 / ((from_nat[Real](n) + Real.1) * (from_nat[Real](n) + Real.1))
//             }, k)
//         },
//         pi * pi / from_nat[Real](Nat.6))
// }
//
// What this file does prove is the convergence machinery for the same series:
// the partial sums above are exactly the Basel partial sums (the term
// functions agree pointwise), and they are bounded by two and converge.

/// The Basel partial sums sum_{n=0}^{k-1} 1 / (n + 1)^2 equal the zeta(2)
/// partial sums.
theorem zeta2_partial_basel_form(k: Nat) {
    partial(function(n: Nat) {
        Real.1 / ((from_nat[Real](n) + Real.1) * (from_nat[Real](n) + Real.1))
    }, k) = partial(zeta2_term, k)
} by {
    forall(n: Nat) {
        if n < k {
            from_nat_add[Real](n, Nat.1)
            from_nat[Real](n + Nat.1) = from_nat[Real](n) + from_nat[Real](Nat.1)
            from_nat_one[Real]
            from_nat[Real](Nat.1) = Real.1
            n + Nat.1 = n.suc
            from_nat[Real](n.suc) = from_nat[Real](n) + Real.1
            Real.1 / ((from_nat[Real](n) + Real.1) * (from_nat[Real](n) + Real.1)) =
                Real.1 / (from_nat[Real](n.suc) * from_nat[Real](n.suc))
            zeta2_term(n) = Real.1 / (from_nat[Real](n.suc) * from_nat[Real](n.suc))
            Real.1 / ((from_nat[Real](n) + Real.1) * (from_nat[Real](n) + Real.1)) = zeta2_term(n)
        }
    }
    partial_pointwise_eq(function(n: Nat) {
        Real.1 / ((from_nat[Real](n) + Real.1) * (from_nat[Real](n) + Real.1))
    }, zeta2_term, k)
    partial(function(n: Nat) {
        Real.1 / ((from_nat[Real](n) + Real.1) * (from_nat[Real](n) + Real.1))
    }, k) = partial(zeta2_term, k)
}

// ---------------------------------------------------------------------------
// The alternating (Mercator) series for ln 2.
//
// The Mercator series is
//
//     ln 2 = sum_{n=1}^{infinity} (-1)^(n + 1) / n = 1 - 1/2 + 1/3 - 1/4 + ...
//
// In the library's indexing (from n = 0, with the alternating sign
// alternating_sign[Real](k) = (-1)^k), the kth summand is
// alt_term(harmonic)(k) = (-1)^k / (k + 1), and the partial sums of
// alt_term(harmonic) are the partial sums of the Mercator series.  The value
// ln 2 needs the log series (the library's log is Option-valued and the value
// of ln 2 is not yet derived), so the equality with ln 2 is stated here as a
// comment; what is proved is that the harmonic terms decrease to zero (so the
// alternating series test applies) and the first few partial sums.
//
// theorem mercator_ln_two {
//     converges_to(partial(alt_term(harmonic)), Real.2.log)
// }

/// The harmonic terms are decreasing: 1 / (n + 2) <= 1 / (n + 1).
theorem harmonic_decreasing {
    is_decreasing_seq(harmonic)
} by {
    forall(n: Nat) {
        n <= n.suc
        nat_recip_antitone_suc(n, n.suc)
        Real.1 / from_nat[Real](n.suc.suc) <= Real.1 / from_nat[Real](n.suc)
        harmonic(n.suc) = Real.1 / from_nat[Real](n.suc.suc)
        harmonic(n) = Real.1 / from_nat[Real](n.suc)
        harmonic(n.suc) <= harmonic(n)
    }
    is_decreasing_seq(harmonic)
}

/// The first two partial sums of the Mercator series: 1 - 1/2 = 1/2.
theorem mercator_partial_two {
    partial(alt_term(harmonic), Nat.2) = Real.1 / from_nat[Real](Nat.2)
} by {
    partial_suc(alt_term(harmonic), Nat.1)
    partial(alt_term(harmonic), Nat.2) = partial(alt_term(harmonic), Nat.1) + alt_term(harmonic)(Nat.1)
    partial_suc(alt_term(harmonic), Nat.0)
    partial(alt_term(harmonic), Nat.1) = partial(alt_term(harmonic), Nat.0) + alt_term(harmonic)(Nat.0)
    partial_zero(alt_term(harmonic))
    partial(alt_term(harmonic), Nat.0) = Real.0
    alt_term(harmonic)(Nat.0) = alternating_sign[Real](Nat.0) * harmonic(Nat.0)
    alternating_sign_zero[Real]
    alternating_sign[Real](Nat.0) = Real.1
    harmonic(Nat.0) = Real.1
    alt_term(harmonic)(Nat.0) = Real.1
    alt_term(harmonic)(Nat.1) = alternating_sign[Real](Nat.1) * harmonic(Nat.1)
    alternating_sign_suc[Real](Nat.0)
    alternating_sign[Real](Nat.1) = -alternating_sign[Real](Nat.0)
    alternating_sign[Real](Nat.1) = -Real.1
    harmonic(Nat.1) = Real.1 / from_nat[Real](Nat.2)
    alt_term(harmonic)(Nat.1) = -Real.1 * (Real.1 / from_nat[Real](Nat.2))
    -Real.1 * (Real.1 / from_nat[Real](Nat.2)) = -(Real.1 / from_nat[Real](Nat.2))
    partial(alt_term(harmonic), Nat.1) = Real.1
    partial(alt_term(harmonic), Nat.2) = Real.1 - Real.1 / from_nat[Real](Nat.2)
    // Two as a real: from_nat(Nat.2) = 1 + 1.
    from_nat_add[Real](Nat.1, Nat.1)
    from_nat[Real](Nat.1 + Nat.1) = from_nat[Real](Nat.1) + from_nat[Real](Nat.1)
    Nat.1 + Nat.1 = Nat.2
    from_nat_one[Real]
    from_nat[Real](Nat.1) = Real.1
    from_nat[Real](Nat.2) = Real.1 + Real.1
    // One is two over two: 1 = 2 / 2.
    from_nat_suc_pos_real(Nat.1)
    from_nat[Real](Nat.2) > Real.0
    from_nat[Real](Nat.2) != Real.0
    div_mul_cancel_left(from_nat[Real](Nat.2), Real.1)
    (from_nat[Real](Nat.2) * Real.1) / from_nat[Real](Nat.2) = Real.1
    from_nat[Real](Nat.2) * Real.1 = from_nat[Real](Nat.2)
    from_nat[Real](Nat.2) / from_nat[Real](Nat.2) = Real.1
    Real.1 = from_nat[Real](Nat.2) / from_nat[Real](Nat.2)
    // Subtract the two unit fractions over the common denominator two.
    div_sub_same_denom(Real.1, from_nat[Real](Nat.2), Real.1, Real.1, from_nat[Real](Nat.2))
    Real.1 * from_nat[Real](Nat.2) / from_nat[Real](Nat.2) - Real.1 * Real.1 / from_nat[Real](Nat.2) =
        (Real.1 * from_nat[Real](Nat.2) - Real.1 * Real.1) / from_nat[Real](Nat.2)
    Real.1 * from_nat[Real](Nat.2) / from_nat[Real](Nat.2) = from_nat[Real](Nat.2) / from_nat[Real](Nat.2)
    Real.1 * Real.1 / from_nat[Real](Nat.2) = Real.1 / from_nat[Real](Nat.2)
    Real.1 - Real.1 / from_nat[Real](Nat.2) = (Real.1 * from_nat[Real](Nat.2) - Real.1 * Real.1) / from_nat[Real](Nat.2)
    // The numerator is one: 1 * 2 - 1 * 1 = 2 - 1 = 1.
    Real.1 * from_nat[Real](Nat.2) = from_nat[Real](Nat.2)
    Real.1 * Real.1 = Real.1
    from_nat[Real](Nat.2) = Real.1 + Real.1
    from_nat[Real](Nat.2) - Real.1 = Real.1
    Real.1 * from_nat[Real](Nat.2) - Real.1 * Real.1 = from_nat[Real](Nat.2) - Real.1
    Real.1 * from_nat[Real](Nat.2) - Real.1 * Real.1 = Real.1
    (Real.1 * from_nat[Real](Nat.2) - Real.1 * Real.1) / from_nat[Real](Nat.2) = Real.1 / from_nat[Real](Nat.2)
    Real.1 - Real.1 / from_nat[Real](Nat.2) = Real.1 / from_nat[Real](Nat.2)
    partial(alt_term(harmonic), Nat.2) = Real.1 / from_nat[Real](Nat.2)
}

/// The first four partial sums of the Mercator series:
/// 1 - 1/2 + 1/3 - 1/4 = 7/12.
theorem mercator_partial_four {
    partial(alt_term(harmonic), Nat.4) = from_nat[Real](Nat.7) / from_nat[Real](Nat.12)
} by {
    // Expand partial(alt_term(harmonic), 4).
    partial_suc(alt_term(harmonic), Nat.3)
    partial(alt_term(harmonic), Nat.4) = partial(alt_term(harmonic), Nat.3) + alt_term(harmonic)(Nat.3)
    partial_suc(alt_term(harmonic), Nat.2)
    partial(alt_term(harmonic), Nat.3) = partial(alt_term(harmonic), Nat.2) + alt_term(harmonic)(Nat.2)
    mercator_partial_two
    partial(alt_term(harmonic), Nat.2) = Real.1 / from_nat[Real](Nat.2)
    // The third summand is 1/3.
    alt_term(harmonic)(Nat.2) = alternating_sign[Real](Nat.2) * harmonic(Nat.2)
    alternating_sign_suc[Real](Nat.1)
    alternating_sign[Real](Nat.2) = -alternating_sign[Real](Nat.1)
    alternating_sign_suc[Real](Nat.0)
    alternating_sign[Real](Nat.1) = -alternating_sign[Real](Nat.0)
    alternating_sign_zero[Real]
    alternating_sign[Real](Nat.0) = Real.1
    alternating_sign[Real](Nat.1) = -Real.1
    alternating_sign[Real](Nat.2) = Real.1
    harmonic(Nat.2) = Real.1 / from_nat[Real](Nat.3)
    alt_term(harmonic)(Nat.2) = Real.1 * (Real.1 / from_nat[Real](Nat.3))
    Real.1 * (Real.1 / from_nat[Real](Nat.3)) = Real.1 / from_nat[Real](Nat.3)
    alt_term(harmonic)(Nat.2) = Real.1 / from_nat[Real](Nat.3)
    partial(alt_term(harmonic), Nat.3) = Real.1 / from_nat[Real](Nat.2) + Real.1 / from_nat[Real](Nat.3)
    // The fourth summand is -1/4.
    alt_term(harmonic)(Nat.3) = alternating_sign[Real](Nat.3) * harmonic(Nat.3)
    alternating_sign_suc[Real](Nat.2)
    alternating_sign[Real](Nat.3) = -alternating_sign[Real](Nat.2)
    alternating_sign[Real](Nat.3) = -Real.1
    harmonic(Nat.3) = Real.1 / from_nat[Real](Nat.4)
    alt_term(harmonic)(Nat.3) = -Real.1 * (Real.1 / from_nat[Real](Nat.4))
    -Real.1 * (Real.1 / from_nat[Real](Nat.4)) = -(Real.1 / from_nat[Real](Nat.4))
    alt_term(harmonic)(Nat.3) = -(Real.1 / from_nat[Real](Nat.4))
    partial(alt_term(harmonic), Nat.4) =
        Real.1 / from_nat[Real](Nat.2) + Real.1 / from_nat[Real](Nat.3) - Real.1 / from_nat[Real](Nat.4)
    // Half plus a third is five sixths: 1/2 + 1/3 = 5/6.
    from_nat_mul[Real](Nat.3, Nat.2)
    from_nat[Real](Nat.3 * Nat.2) = from_nat[Real](Nat.3) * from_nat[Real](Nat.2)
    Nat.3 * Nat.2 = Nat.6
    from_nat[Real](Nat.6) = from_nat[Real](Nat.3) * from_nat[Real](Nat.2)
    from_nat_suc_pos_real(Nat.2)
    from_nat[Real](Nat.3) != Real.0
    from_nat_suc_pos_real(Nat.1)
    from_nat[Real](Nat.2) != Real.0
    div_cancel_common(Real.1, from_nat[Real](Nat.3), from_nat[Real](Nat.2))
    (Real.1 * from_nat[Real](Nat.3)) / (from_nat[Real](Nat.2) * from_nat[Real](Nat.3)) = Real.1 / from_nat[Real](Nat.2)
    from_nat[Real](Nat.2) * from_nat[Real](Nat.3) = from_nat[Real](Nat.6)
    Real.1 * from_nat[Real](Nat.3) = from_nat[Real](Nat.3)
    from_nat[Real](Nat.3) / from_nat[Real](Nat.6) = Real.1 / from_nat[Real](Nat.2)
    Real.1 / from_nat[Real](Nat.2) = from_nat[Real](Nat.3) / from_nat[Real](Nat.6)
    from_nat_suc_pos_real(Nat.0)
    from_nat[Real](Nat.1) != Real.0
    div_cancel_common(Real.1, from_nat[Real](Nat.2), from_nat[Real](Nat.3))
    (Real.1 * from_nat[Real](Nat.2)) / (from_nat[Real](Nat.3) * from_nat[Real](Nat.2)) = Real.1 / from_nat[Real](Nat.3)
    from_nat[Real](Nat.3) * from_nat[Real](Nat.2) = from_nat[Real](Nat.6)
    Real.1 * from_nat[Real](Nat.2) = from_nat[Real](Nat.2)
    from_nat[Real](Nat.2) / from_nat[Real](Nat.6) = Real.1 / from_nat[Real](Nat.3)
    Real.1 / from_nat[Real](Nat.3) = from_nat[Real](Nat.2) / from_nat[Real](Nat.6)
    from_nat_suc_pos_real(Nat.5)
    from_nat[Real](Nat.6) != Real.0
    div_add_same_denom(Real.1, from_nat[Real](Nat.3), from_nat[Real](Nat.2), from_nat[Real](Nat.6))
    Real.1 * from_nat[Real](Nat.3) / from_nat[Real](Nat.6) + Real.1 * from_nat[Real](Nat.2) / from_nat[Real](Nat.6) =
        Real.1 * (from_nat[Real](Nat.3) + from_nat[Real](Nat.2)) / from_nat[Real](Nat.6)
    Real.1 * from_nat[Real](Nat.3) / from_nat[Real](Nat.6) = from_nat[Real](Nat.3) / from_nat[Real](Nat.6)
    Real.1 * from_nat[Real](Nat.2) / from_nat[Real](Nat.6) = from_nat[Real](Nat.2) / from_nat[Real](Nat.6)
    from_nat_add[Real](Nat.3, Nat.2)
    from_nat[Real](Nat.3 + Nat.2) = from_nat[Real](Nat.3) + from_nat[Real](Nat.2)
    Nat.3 + Nat.2 = Nat.5
    from_nat[Real](Nat.5) = from_nat[Real](Nat.3) + from_nat[Real](Nat.2)
    Real.1 * (from_nat[Real](Nat.3) + from_nat[Real](Nat.2)) = from_nat[Real](Nat.5)
    Real.1 * (from_nat[Real](Nat.3) + from_nat[Real](Nat.2)) / from_nat[Real](Nat.6) = from_nat[Real](Nat.5) / from_nat[Real](Nat.6)
    Real.1 / from_nat[Real](Nat.2) + Real.1 / from_nat[Real](Nat.3) = from_nat[Real](Nat.5) / from_nat[Real](Nat.6)
    partial(alt_term(harmonic), Nat.4) = from_nat[Real](Nat.5) / from_nat[Real](Nat.6) - Real.1 / from_nat[Real](Nat.4)
    // Five sixths minus a quarter is seven twelfths: 5/6 - 1/4 = 7/12.
    from_nat_mul[Real](Nat.6, Nat.2)
    from_nat[Real](Nat.6 * Nat.2) = from_nat[Real](Nat.6) * from_nat[Real](Nat.2)
    Nat.6 * Nat.2 = Nat.12
    from_nat[Real](Nat.12) = from_nat[Real](Nat.6) * from_nat[Real](Nat.2)
    from_nat_suc_pos_real(Nat.5)
    from_nat[Real](Nat.6) != Real.0
    div_cancel_common(from_nat[Real](Nat.5), from_nat[Real](Nat.2), from_nat[Real](Nat.6))
    (from_nat[Real](Nat.5) * from_nat[Real](Nat.2)) / (from_nat[Real](Nat.6) * from_nat[Real](Nat.2)) = from_nat[Real](Nat.5) / from_nat[Real](Nat.6)
    from_nat[Real](Nat.6) * from_nat[Real](Nat.2) = from_nat[Real](Nat.12)
    from_nat_mul[Real](Nat.5, Nat.2)
    from_nat[Real](Nat.5 * Nat.2) = from_nat[Real](Nat.5) * from_nat[Real](Nat.2)
    Nat.5 * Nat.2 = Nat.10
    from_nat[Real](Nat.10) = from_nat[Real](Nat.5) * from_nat[Real](Nat.2)
    from_nat[Real](Nat.5) * from_nat[Real](Nat.2) = from_nat[Real](Nat.10)
    from_nat[Real](Nat.10) / from_nat[Real](Nat.12) = from_nat[Real](Nat.5) / from_nat[Real](Nat.6)
    from_nat[Real](Nat.5) / from_nat[Real](Nat.6) = from_nat[Real](Nat.10) / from_nat[Real](Nat.12)
    from_nat_suc_pos_real(Nat.3)
    from_nat[Real](Nat.4) != Real.0
    div_cancel_common(Real.1, from_nat[Real](Nat.3), from_nat[Real](Nat.4))
    (Real.1 * from_nat[Real](Nat.3)) / (from_nat[Real](Nat.4) * from_nat[Real](Nat.3)) = Real.1 / from_nat[Real](Nat.4)
    from_nat_mul[Real](Nat.4, Nat.3)
    from_nat[Real](Nat.4 * Nat.3) = from_nat[Real](Nat.4) * from_nat[Real](Nat.3)
    Nat.4 * Nat.3 = Nat.12
    from_nat[Real](Nat.12) = from_nat[Real](Nat.4) * from_nat[Real](Nat.3)
    from_nat[Real](Nat.4) * from_nat[Real](Nat.3) = from_nat[Real](Nat.12)
    Real.1 * from_nat[Real](Nat.3) = from_nat[Real](Nat.3)
    from_nat[Real](Nat.3) / from_nat[Real](Nat.12) = Real.1 / from_nat[Real](Nat.4)
    Real.1 / from_nat[Real](Nat.4) = from_nat[Real](Nat.3) / from_nat[Real](Nat.12)
    mul_not_zero[Real](from_nat[Real](Nat.6), from_nat[Real](Nat.2))
    from_nat[Real](Nat.6) * from_nat[Real](Nat.2) != Real.0
    from_nat[Real](Nat.12) != Real.0
    div_sub_same_denom(Real.1, from_nat[Real](Nat.10), Real.1, from_nat[Real](Nat.3), from_nat[Real](Nat.12))
    Real.1 * from_nat[Real](Nat.10) / from_nat[Real](Nat.12) - Real.1 * from_nat[Real](Nat.3) / from_nat[Real](Nat.12) =
        (Real.1 * from_nat[Real](Nat.10) - Real.1 * from_nat[Real](Nat.3)) / from_nat[Real](Nat.12)
    Real.1 * from_nat[Real](Nat.10) / from_nat[Real](Nat.12) = from_nat[Real](Nat.10) / from_nat[Real](Nat.12)
    Real.1 * from_nat[Real](Nat.3) / from_nat[Real](Nat.12) = from_nat[Real](Nat.3) / from_nat[Real](Nat.12)
    Real.1 * from_nat[Real](Nat.10) = from_nat[Real](Nat.10)
    Real.1 * from_nat[Real](Nat.3) = from_nat[Real](Nat.3)
    from_nat_add[Real](Nat.7, Nat.3)
    from_nat[Real](Nat.7 + Nat.3) = from_nat[Real](Nat.7) + from_nat[Real](Nat.3)
    add_suc_right(Nat.7, Nat.2)
    add_suc_right(Nat.7, Nat.1)
    add_one_right(Nat.7)
    from_nat[Real](Nat.10) = from_nat[Real](Nat.7) + from_nat[Real](Nat.3)
    from_nat[Real](Nat.10) - from_nat[Real](Nat.3) = from_nat[Real](Nat.7)
    Real.1 * from_nat[Real](Nat.10) - Real.1 * from_nat[Real](Nat.3) = from_nat[Real](Nat.7)
    (Real.1 * from_nat[Real](Nat.10) - Real.1 * from_nat[Real](Nat.3)) / from_nat[Real](Nat.12) = from_nat[Real](Nat.7) / from_nat[Real](Nat.12)
    from_nat[Real](Nat.5) / from_nat[Real](Nat.6) - Real.1 / from_nat[Real](Nat.4) = from_nat[Real](Nat.7) / from_nat[Real](Nat.12)
    partial(alt_term(harmonic), Nat.4) = from_nat[Real](Nat.7) / from_nat[Real](Nat.12)
}

// ---------------------------------------------------------------------------
// The value of zeta(4).
//
// The next classical value is
//
//     zeta(4) = sum_{n=1}^{infinity} 1 / n^4 = pi^4 / 90,
//
// proved by Euler with the same sine-product method as the Basel problem
// (or by the Fourier series of x^4 and Parseval's identity).  The library
// does not yet have the machinery for the fourth-power series or for pi^4,
// so the statement is recorded here as a comment.
//
// theorem zeta_four_pi_four_over_ninety {
//     converges_to(
//         function(k: Nat) {
//             partial(function(n: Nat) {
//                 Real.1 / ((from_nat[Real](n) + Real.1).pow(Nat.4))
//             }, k)
//         },
//         pi.pow(Nat.4) / from_nat[Real](Nat.90))
// }
