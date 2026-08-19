/// Newton's method for real roots.
///
/// Newton's method for a real function f with derivative f' iterates the map
/// x ↦ x - f(x)/f'(x), starting from an initial guess x0.  A simple root of f
/// (a point where f vanishes and f' does not) is a fixed point of the map,
/// and the error of the iteration transforms quadratically: the error of the
/// next iterate equals the square of the current error times a factor that
/// stays bounded when the second derivative is bounded and the first
/// derivative is bounded away from zero.  This file proves the fixed-point
/// property, the quadratic error transformation (via Taylor's theorem of
/// order two with Lagrange remainder), and states the local convergence
/// theorem.

from order import lt_trans, lt_imp_lte, lte_refl, lte_trans
from nat import Nat, alt_induction
from real.continuity_base import Real
from real.calculus_api import is_derivative_fn
from real.real_base import add_comm, add_assoc, add_zero_right, neg_zero, neg_neg, neg_distrib, sub_cancels, sub_moves_sides, lt_add_pos, pos_gt_zero, gt_zero_imp_pos, pos_imp_eq_abs, abs_gte_zero
from real.real_ring import mul_zero_left, real_mul_comm, mul_assoc, mul_neg_left, mul_sub_distrib_left, mul_abs, mul_pos_pos, mul_nonneg, pos_lte_imp_pos
from real.real_field import div_mul_cancel_left, mul_le_mul_pos_right
from real.real_seq import lt_imp_minus_pos
from real.taylor import taylor_order_two, taylor_two, taylor2_sub_add_cancel
from real.lhopital import sub_sub_distrib
from real.exp import abs_div
from real.harmonic import real_recip_antitone_pos
from ordered_field import inverse_of_positive_is_positive, multiply_inequality_with_nonnegative_element

numerals Real

/// The Newton step map: x ↦ x - f(x) / f'(x).
define newton_step(f: Real -> Real, fp: Real -> Real, x: Real) -> Real {
    x - f(x) / fp(x)
}

/// The Newton iteration: the sequence x_{n+1} = newton_step(x_n) with x_0 = x0.
define newton_seq(f: Real -> Real, fp: Real -> Real, x0: Real, n: Nat) -> Real {
    match n {
        Nat.zero {
            x0
        }
        Nat.suc(m) {
            newton_step(f, fp, newton_seq(f, fp, x0, m))
        }
    }
}

/// The first Newton iterate is the initial guess.
theorem newton_seq_zero(f: Real -> Real, fp: Real -> Real, x0: Real) {
    newton_seq(f, fp, x0, Nat.0) = x0
} by {
    match Nat.0 {
        Nat.zero {
            newton_seq(f, fp, x0, Nat.0) = x0
        }
        Nat.suc(k) {
            false
        }
    }
}

/// Each Newton iterate is the Newton step applied to the previous one.
theorem newton_seq_suc(f: Real -> Real, fp: Real -> Real, x0: Real, n: Nat) {
    newton_seq(f, fp, x0, n.suc) = newton_step(f, fp, newton_seq(f, fp, x0, n))
} by {
    match n.suc {
        Nat.zero {
            false
        }
        Nat.suc(k) {
            k = n
            newton_seq(f, fp, x0, n.suc) = newton_step(f, fp, newton_seq(f, fp, x0, n))
        }
    }
}

/// A simple root is a fixed point of the Newton step.
theorem newton_step_root_fixed_point(f: Real -> Real, fp: Real -> Real, xstar: Real) {
    f(xstar) = Real.0 and fp(xstar) != Real.0 implies
        newton_step(f, fp, xstar) = xstar
} by {
    if f(xstar) = Real.0 and fp(xstar) != Real.0 {
        newton_step(f, fp, xstar) = xstar - f(xstar) / fp(xstar)
        f(xstar) = Real.0
        xstar - f(xstar) / fp(xstar) = xstar - Real.0 / fp(xstar)
        Real.0 / fp(xstar) = Real.0 * fp(xstar).inverse
        mul_zero_left(fp(xstar).inverse)
        Real.0 * fp(xstar).inverse = Real.0
        Real.0 / fp(xstar) = Real.0
        xstar - Real.0 / fp(xstar) = xstar - Real.0
        xstar - Real.0 = xstar + -Real.0
        neg_zero
        -Real.0 = Real.0
        xstar + -Real.0 = xstar + Real.0
        add_zero_right(xstar)
        xstar + Real.0 = xstar
        xstar - Real.0 = xstar
        newton_step(f, fp, xstar) = xstar
    }
}

/// Starting the Newton iteration at a simple root keeps every iterate at the root.
theorem newton_seq_root_constant(f: Real -> Real, fp: Real -> Real, xstar: Real) {
    f(xstar) = Real.0 and fp(xstar) != Real.0 implies
        forall(n: Nat) { newton_seq(f, fp, xstar, n) = xstar }
} by {
    if f(xstar) = Real.0 and fp(xstar) != Real.0 {
        define p(k: Nat) -> Bool {
            newton_seq(f, fp, xstar, k) = xstar
        }
        newton_seq(f, fp, xstar, Nat.0) = xstar
        p(Nat.0)
        forall(k: Nat) {
            if p(k) {
                p(k) = (newton_seq(f, fp, xstar, k) = xstar)
                newton_seq(f, fp, xstar, k) = xstar
                newton_seq(f, fp, xstar, k.suc) =
                    newton_step(f, fp, newton_seq(f, fp, xstar, k))
                newton_step(f, fp, newton_seq(f, fp, xstar, k)) =
                    newton_step(f, fp, xstar)
                newton_step_root_fixed_point(f, fp, xstar)
                newton_step(f, fp, xstar) = xstar
                newton_seq(f, fp, xstar, k.suc) = xstar
                p(k.suc)
            }
        }
        p(Nat.0) and forall(k: Nat) { p(k) implies p(k.suc) }
        alt_induction(p)
        forall(k: Nat) { p(k) }
        forall(n: Nat) { newton_seq(f, fp, xstar, n) = xstar }
    }
}

/// The quadratic error transformation: if x* is a simple root of f and x < x*,
/// the error of one Newton step equals the square of the current error times a
/// second-derivative factor evaluated at an interior point.  This is Taylor's
/// theorem of order two with Lagrange remainder, applied at x and evaluated
/// at x*.
theorem newton_error_quadratic(f: Real -> Real, fp: Real -> Real, ddf: Real -> Real, x: Real, xstar: Real) {
    x < xstar and f(xstar) = Real.0 and fp(x) != Real.0 and
    is_derivative_fn(f, fp) and is_derivative_fn(fp, ddf)
    implies exists(c: Real) {
        x < c and c < xstar and
        newton_step(f, fp, x) - xstar =
            ((ddf(c) / taylor_two) * (xstar - x) * (xstar - x)) / fp(x)
    }
} by {
    if x < xstar and f(xstar) = Real.0 and fp(x) != Real.0 and
       is_derivative_fn(f, fp) and is_derivative_fn(fp, ddf) {
        taylor_order_two(f, fp, ddf, x, xstar)
        let c: Real satisfy {
            x < c and c < xstar and
            f(xstar) = f(x) + fp(x) * (xstar - x) +
                (ddf(c) / taylor_two) * (xstar - x) * (xstar - x)
        }
        x < c and c < xstar
        f(xstar) = f(x) + fp(x) * (xstar - x) +
            (ddf(c) / taylor_two) * (xstar - x) * (xstar - x)
        f(xstar) = Real.0
        f(x) + fp(x) * (xstar - x) +
            (ddf(c) / taylor_two) * (xstar - x) * (xstar - x) = Real.0
        add_assoc(f(x), fp(x) * (xstar - x),
            (ddf(c) / taylor_two) * (xstar - x) * (xstar - x))
        f(x) + (fp(x) * (xstar - x) +
            (ddf(c) / taylor_two) * (xstar - x) * (xstar - x)) = Real.0
        sub_moves_sides(f(x), fp(x) * (xstar - x) +
            (ddf(c) / taylor_two) * (xstar - x) * (xstar - x), Real.0)
        f(x) = Real.0 - (fp(x) * (xstar - x) +
            (ddf(c) / taylor_two) * (xstar - x) * (xstar - x))
        Real.0 - (fp(x) * (xstar - x) +
            (ddf(c) / taylor_two) * (xstar - x) * (xstar - x)) =
            -(fp(x) * (xstar - x) +
                (ddf(c) / taylor_two) * (xstar - x) * (xstar - x))
        neg_distrib(fp(x) * (xstar - x),
            (ddf(c) / taylor_two) * (xstar - x) * (xstar - x))
        -(fp(x) * (xstar - x) +
            (ddf(c) / taylor_two) * (xstar - x) * (xstar - x)) =
            -(fp(x) * (xstar - x)) +
                -((ddf(c) / taylor_two) * (xstar - x) * (xstar - x))
        -(fp(x) * (xstar - x)) +
            -((ddf(c) / taylor_two) * (xstar - x) * (xstar - x)) =
            -(fp(x) * (xstar - x)) -
                (ddf(c) / taylor_two) * (xstar - x) * (xstar - x)
        f(x) = -(fp(x) * (xstar - x)) -
            (ddf(c) / taylor_two) * (xstar - x) * (xstar - x)
        f(x) / fp(x) =
            (-(fp(x) * (xstar - x)) -
                (ddf(c) / taylor_two) * (xstar - x) * (xstar - x)) / fp(x)
        (-(fp(x) * (xstar - x)) -
            (ddf(c) / taylor_two) * (xstar - x) * (xstar - x)) / fp(x) =
            (-(fp(x) * (xstar - x)) -
                (ddf(c) / taylor_two) * (xstar - x) * (xstar - x)) * fp(x).inverse
        mul_sub_distrib_left(-(fp(x) * (xstar - x)),
            (ddf(c) / taylor_two) * (xstar - x) * (xstar - x), fp(x).inverse)
        (-(fp(x) * (xstar - x)) -
            (ddf(c) / taylor_two) * (xstar - x) * (xstar - x)) * fp(x).inverse =
            (-(fp(x) * (xstar - x))) * fp(x).inverse -
                ((ddf(c) / taylor_two) * (xstar - x) * (xstar - x)) * fp(x).inverse
        (-(fp(x) * (xstar - x))) * fp(x).inverse =
            (-(fp(x) * (xstar - x))) / fp(x)
        ((ddf(c) / taylor_two) * (xstar - x) * (xstar - x)) * fp(x).inverse =
            ((ddf(c) / taylor_two) * (xstar - x) * (xstar - x)) / fp(x)
        (-(fp(x) * (xstar - x)) -
            (ddf(c) / taylor_two) * (xstar - x) * (xstar - x)) / fp(x) =
            (-(fp(x) * (xstar - x))) / fp(x) -
                ((ddf(c) / taylor_two) * (xstar - x) * (xstar - x)) / fp(x)
        (-(fp(x) * (xstar - x))) / fp(x) =
            -(fp(x) * (xstar - x)) * fp(x).inverse
        mul_neg_left(fp(x) * (xstar - x), fp(x).inverse)
        -(fp(x) * (xstar - x)) * fp(x).inverse =
            -((fp(x) * (xstar - x)) * fp(x).inverse)
        -((fp(x) * (xstar - x)) * fp(x).inverse) =
            -((fp(x) * (xstar - x)) / fp(x))
        (-(fp(x) * (xstar - x))) / fp(x) =
            -((fp(x) * (xstar - x)) / fp(x))
        div_mul_cancel_left(fp(x), xstar - x)
        (fp(x) * (xstar - x)) / fp(x) = xstar - x
        (-(fp(x) * (xstar - x))) / fp(x) = -(xstar - x)
        f(x) / fp(x) = -(xstar - x) -
            ((ddf(c) / taylor_two) * (xstar - x) * (xstar - x)) / fp(x)
        newton_step(f, fp, x) = x - f(x) / fp(x)
        newton_step(f, fp, x) - xstar = x - f(x) / fp(x) - xstar
        x - f(x) / fp(x) - xstar =
            x - (-(xstar - x) -
                ((ddf(c) / taylor_two) * (xstar - x) * (xstar - x)) / fp(x)) - xstar
        sub_sub_distrib(x, -(xstar - x),
            ((ddf(c) / taylor_two) * (xstar - x) * (xstar - x)) / fp(x))
        x - (-(xstar - x) -
            ((ddf(c) / taylor_two) * (xstar - x) * (xstar - x)) / fp(x)) =
            x - (-(xstar - x)) +
                ((ddf(c) / taylor_two) * (xstar - x) * (xstar - x)) / fp(x)
        neg_neg(xstar - x)
        -(-(xstar - x)) = xstar - x
        x - (-(xstar - x)) = x + -(-(xstar - x))
        x + -(-(xstar - x)) = x + (xstar - x)
        x - (-(xstar - x)) = x + (xstar - x)
        x - (-(xstar - x)) +
            ((ddf(c) / taylor_two) * (xstar - x) * (xstar - x)) / fp(x) =
            x + (xstar - x) +
                ((ddf(c) / taylor_two) * (xstar - x) * (xstar - x)) / fp(x)
        add_comm(x, xstar - x)
        x + (xstar - x) = (xstar - x) + x
        taylor2_sub_add_cancel(xstar, x)
        (xstar - x) + x = xstar
        x + (xstar - x) = xstar
        x + (xstar - x) +
            ((ddf(c) / taylor_two) * (xstar - x) * (xstar - x)) / fp(x) =
            xstar + ((ddf(c) / taylor_two) * (xstar - x) * (xstar - x)) / fp(x)
        x - (-(xstar - x) -
            ((ddf(c) / taylor_two) * (xstar - x) * (xstar - x)) / fp(x)) =
            xstar + ((ddf(c) / taylor_two) * (xstar - x) * (xstar - x)) / fp(x)
        x - (-(xstar - x) -
            ((ddf(c) / taylor_two) * (xstar - x) * (xstar - x)) / fp(x)) - xstar =
            xstar + ((ddf(c) / taylor_two) * (xstar - x) * (xstar - x)) / fp(x) - xstar
        add_comm(xstar, ((ddf(c) / taylor_two) * (xstar - x) * (xstar - x)) / fp(x))
        xstar + ((ddf(c) / taylor_two) * (xstar - x) * (xstar - x)) / fp(x) =
            ((ddf(c) / taylor_two) * (xstar - x) * (xstar - x)) / fp(x) + xstar
        sub_cancels(((ddf(c) / taylor_two) * (xstar - x) * (xstar - x)) / fp(x), xstar)
        ((ddf(c) / taylor_two) * (xstar - x) * (xstar - x)) / fp(x) + xstar - xstar =
            ((ddf(c) / taylor_two) * (xstar - x) * (xstar - x)) / fp(x)
        xstar + ((ddf(c) / taylor_two) * (xstar - x) * (xstar - x)) / fp(x) - xstar =
            ((ddf(c) / taylor_two) * (xstar - x) * (xstar - x)) / fp(x)
        x - (-(xstar - x) -
            ((ddf(c) / taylor_two) * (xstar - x) * (xstar - x)) / fp(x)) - xstar =
            ((ddf(c) / taylor_two) * (xstar - x) * (xstar - x)) / fp(x)
        x - f(x) / fp(x) - xstar =
            ((ddf(c) / taylor_two) * (xstar - x) * (xstar - x)) / fp(x)
        newton_step(f, fp, x) - xstar =
            ((ddf(c) / taylor_two) * (xstar - x) * (xstar - x)) / fp(x)
        exists(c2: Real) {
            x < c2 and c2 < xstar and
            newton_step(f, fp, x) - xstar =
                ((ddf(c2) / taylor_two) * (xstar - x) * (xstar - x)) / fp(x)
        }
    }
}

/// The real number two (1 + 1) is positive.
theorem newton_taylor_two_pos {
    taylor_two > Real.0
} by {
    Real.1.is_positive
    pos_gt_zero(Real.1)
    Real.1 > Real.0
    lt_add_pos(Real.1, Real.1)
    Real.1 < Real.1 + Real.1
    taylor_two = Real.1 + Real.1
    lt_trans(Real.0, Real.1, Real.1 + Real.1)
    Real.0 < Real.1 + Real.1
    Real.0 < taylor_two
    taylor_two > Real.0
}

/// The error of the Newton iteration transforms quadratically at every step
/// that stays left of the simple root.
theorem newton_seq_error_quadratic(f: Real -> Real, fp: Real -> Real, ddf: Real -> Real, x0: Real, xstar: Real, n: Nat) {
    newton_seq(f, fp, x0, n) < xstar and f(xstar) = Real.0 and
    fp(newton_seq(f, fp, x0, n)) != Real.0 and
    is_derivative_fn(f, fp) and is_derivative_fn(fp, ddf)
    implies exists(c: Real) {
        newton_seq(f, fp, x0, n) < c and c < xstar and
        newton_seq(f, fp, x0, n.suc) - xstar =
            ((ddf(c) / taylor_two) * (xstar - newton_seq(f, fp, x0, n)) *
                (xstar - newton_seq(f, fp, x0, n))) /
                fp(newton_seq(f, fp, x0, n))
    }
} by {
    if newton_seq(f, fp, x0, n) < xstar and f(xstar) = Real.0 and
       fp(newton_seq(f, fp, x0, n)) != Real.0 and
       is_derivative_fn(f, fp) and is_derivative_fn(fp, ddf) {
        newton_error_quadratic(f, fp, ddf, newton_seq(f, fp, x0, n), xstar)
        let c: Real satisfy {
            newton_seq(f, fp, x0, n) < c and c < xstar and
            newton_step(f, fp, newton_seq(f, fp, x0, n)) - xstar =
                ((ddf(c) / taylor_two) * (xstar - newton_seq(f, fp, x0, n)) *
                    (xstar - newton_seq(f, fp, x0, n))) /
                    fp(newton_seq(f, fp, x0, n))
        }
        newton_seq(f, fp, x0, n) < c and c < xstar
        newton_step(f, fp, newton_seq(f, fp, x0, n)) - xstar =
            ((ddf(c) / taylor_two) * (xstar - newton_seq(f, fp, x0, n)) *
                (xstar - newton_seq(f, fp, x0, n))) /
                fp(newton_seq(f, fp, x0, n))
        newton_seq_suc(f, fp, x0, n)
        newton_seq(f, fp, x0, n.suc) = newton_step(f, fp, newton_seq(f, fp, x0, n))
        newton_seq(f, fp, x0, n.suc) - xstar =
            ((ddf(c) / taylor_two) * (xstar - newton_seq(f, fp, x0, n)) *
                (xstar - newton_seq(f, fp, x0, n))) /
                fp(newton_seq(f, fp, x0, n))
        exists(c2: Real) {
            newton_seq(f, fp, x0, n) < c2 and c2 < xstar and
            newton_seq(f, fp, x0, n.suc) - xstar =
                ((ddf(c2) / taylor_two) * (xstar - newton_seq(f, fp, x0, n)) *
                    (xstar - newton_seq(f, fp, x0, n))) /
                    fp(newton_seq(f, fp, x0, n))
        }
    }
}

/// Local convergence of Newton's method.
///
/// The classical theorem: if x* is a simple root of f, f is twice
/// differentiable, f' is bounded away from zero and f'' is bounded on the
/// interval between x* and the initial guess x0, and x0 is close enough to
/// x*, then the Newton iteration converges to x*.
///
/// The full proof needs three ingredients.  (1) The quadratic error bound of
/// `newton_error_bounded` applied at every iterate, which requires an
/// induction showing the iteration stays left of the root and that the
/// derivative stays bounded away from zero.  (2) The geometric-error lemma:
/// a sequence of errors with |e_{n+1}| <= C |e_n|^2 and C |e_0| < 1 satisfies
/// |e_n| <= C^{-1} (C |e_0|)^{2^n}, which tends to zero.  (3) The transport
/// of the vanishing error back to the convergence of the sequence itself via
/// the sequence-limit API of `real_seq`.  These are all standard, but the
/// full development is long; the statement is recorded here and the proof is
/// left for future work.
// theorem newton_converges(f: Real -> Real, fp: Real -> Real, ddf: Real -> Real,
//     x0: Real, xstar: Real, m: Real, M: Real) {
//     f(xstar) = Real.0 and fp(xstar) != Real.0 and
//     is_derivative_fn(f, fp) and is_derivative_fn(fp, ddf) and
//     x0 < xstar and m.is_positive and M.is_positive and
//     (forall(z: Real) { x0 <= z and z <= xstar implies m <= fp(z).abs }) and
//     (forall(z: Real) { x0 <= z and z <= xstar implies ddf(z).abs <= M }) and
//     (M / (taylor_two * m)) * (xstar - x0) * (xstar - x0) < xstar - x0
//     implies converges_to(newton_seq(f, fp, x0), xstar)
// } by {
//     // See the comment above: this proof requires the full induction that
//     // the iterates stay left of the root, the geometric-error lemma
//     // |e_n| <= C^{-1} (C |e_0|)^{2^n}, and the limit transport.
// }

/// The bounded quadratic error estimate: if the second derivative is bounded
/// by M and the first derivative is bounded below by m in absolute value on
/// [x, x*], the error of one Newton step is at most (M/(2 m)) times the
/// square of the current error.  This is the concrete form of the statement
/// that the error transforms quadratically with a bounded factor.
theorem newton_error_bounded(f: Real -> Real, fp: Real -> Real, ddf: Real -> Real, x: Real, xstar: Real, m: Real, bigm: Real) {
    x < xstar and f(xstar) = Real.0 and fp(x) != Real.0 and
    is_derivative_fn(f, fp) and is_derivative_fn(fp, ddf) and
    m.is_positive and m <= fp(x).abs and
    (forall(z: Real) { x <= z and z <= xstar implies ddf(z).abs <= bigm })
    implies (newton_step(f, fp, x) - xstar).abs <=
        ((bigm / taylor_two) * (xstar - x) * (xstar - x)) / m
} by {
    if x < xstar and f(xstar) = Real.0 and fp(x) != Real.0 and
       is_derivative_fn(f, fp) and is_derivative_fn(fp, ddf) and
       m.is_positive and m <= fp(x).abs and
       (forall(z: Real) { x <= z and z <= xstar implies ddf(z).abs <= bigm }) {
        newton_error_quadratic(f, fp, ddf, x, xstar)
        let c: Real satisfy {
            x < c and c < xstar and
            newton_step(f, fp, x) - xstar =
                ((ddf(c) / taylor_two) * (xstar - x) * (xstar - x)) / fp(x)
        }
        x < c and c < xstar
        newton_step(f, fp, x) - xstar =
            ((ddf(c) / taylor_two) * (xstar - x) * (xstar - x)) / fp(x)
        // The absolute value of the error splits across the division.
        abs_div((ddf(c) / taylor_two) * (xstar - x) * (xstar - x), fp(x))
        (((ddf(c) / taylor_two) * (xstar - x) * (xstar - x)) / fp(x)).abs =
            ((ddf(c) / taylor_two) * (xstar - x) * (xstar - x)).abs / fp(x).abs
        (newton_step(f, fp, x) - xstar).abs =
            ((ddf(c) / taylor_two) * (xstar - x) * (xstar - x)).abs / fp(x).abs
        // The absolute value splits across the product.
        mul_abs(ddf(c) / taylor_two, (xstar - x) * (xstar - x))
        ((ddf(c) / taylor_two) * (xstar - x) * (xstar - x)).abs =
            (ddf(c) / taylor_two).abs * ((xstar - x) * (xstar - x)).abs
        // |ddf(c) / taylor_two| = |ddf(c)| / taylor_two.
        abs_div(ddf(c), taylor_two)
        (ddf(c) / taylor_two).abs = ddf(c).abs / taylor_two.abs
        newton_taylor_two_pos
        taylor_two > Real.0
        pos_imp_eq_abs(taylor_two)
        taylor_two = taylor_two.abs
        ddf(c).abs / taylor_two.abs = ddf(c).abs / taylor_two
        (ddf(c) / taylor_two).abs = ddf(c).abs / taylor_two
        // |(xstar - x)^2| = (xstar - x)^2, since x < xstar.
        lt_imp_minus_pos(x, xstar)
        (xstar - x).is_positive
        pos_gt_zero(xstar - x)
        xstar - x > Real.0
        mul_pos_pos(xstar - x, xstar - x)
        ((xstar - x) * (xstar - x)).is_positive
        pos_gt_zero((xstar - x) * (xstar - x))
        (xstar - x) * (xstar - x) > Real.0
        pos_imp_eq_abs((xstar - x) * (xstar - x))
        (xstar - x) * (xstar - x) = ((xstar - x) * (xstar - x)).abs
        ((xstar - x) * (xstar - x)).abs = (xstar - x) * (xstar - x)
        // The error equals |ddf(c)|/taylor_two * (xstar-x)^2 / |fp(x)|.
        ((ddf(c) / taylor_two) * (xstar - x) * (xstar - x)).abs =
            (ddf(c).abs / taylor_two) * ((xstar - x) * (xstar - x))
        (newton_step(f, fp, x) - xstar).abs =
            ((ddf(c).abs / taylor_two) * ((xstar - x) * (xstar - x))) / fp(x).abs
        // c lies in [x, xstar], so |ddf(c)| <= M.
        lt_imp_lte(x, c)
        x <= c
        lt_imp_lte(c, xstar)
        c <= xstar
        forall(z: Real) { x <= z and z <= xstar implies ddf(z).abs <= bigm }
        x <= c and c <= xstar implies ddf(c).abs <= bigm
        ddf(c).abs <= bigm
        // taylor_two.inverse is positive.
        gt_zero_imp_pos(taylor_two)
        taylor_two.is_positive
        inverse_of_positive_is_positive(taylor_two)
        taylor_two.inverse.is_positive
        pos_gt_zero(taylor_two.inverse)
        taylor_two.inverse > Real.0
        // |ddf(c)| / taylor_two <= bigm / taylor_two.
        mul_le_mul_pos_right(ddf(c).abs, bigm, taylor_two.inverse)
        ddf(c).abs * taylor_two.inverse <= bigm * taylor_two.inverse
        ddf(c).abs / taylor_two = ddf(c).abs * taylor_two.inverse
        ddf(c).abs / taylor_two <= bigm * taylor_two.inverse
        bigm / taylor_two = bigm * taylor_two.inverse
        ddf(c).abs / taylor_two <= bigm / taylor_two
        // The squared distance (xstar - x)^2 is nonnegative.
        lt_imp_lte(Real.0, (xstar - x) * (xstar - x))
        Real.0 <= (xstar - x) * (xstar - x)
        // (|ddf(c)|/taylor_two) * (xstar-x)^2 <= (M/taylor_two) * (xstar-x)^2.
        multiply_inequality_with_nonnegative_element[Real](ddf(c).abs / taylor_two,
            bigm / taylor_two, (xstar - x) * (xstar - x))
        (ddf(c).abs / taylor_two) * ((xstar - x) * (xstar - x)) <= (bigm / taylor_two) * ((xstar - x) * (xstar - x))
        // |fp(x)| is positive and its inverse is positive.
        pos_lte_imp_pos(m, fp(x).abs)
        fp(x).abs.is_positive
        pos_gt_zero(fp(x).abs)
        fp(x).abs > Real.0
        inverse_of_positive_is_positive(fp(x).abs)
        fp(x).abs.inverse.is_positive
        pos_gt_zero(fp(x).abs.inverse)
        fp(x).abs.inverse > Real.0
        // Dividing by |fp(x)| preserves the inequality.
        mul_le_mul_pos_right((ddf(c).abs / taylor_two) * ((xstar - x) * (xstar - x)),
            (bigm / taylor_two) * ((xstar - x) * (xstar - x)), fp(x).abs.inverse)
        ((ddf(c).abs / taylor_two) * ((xstar - x) * (xstar - x))) * fp(x).abs.inverse <= ((bigm / taylor_two) * ((xstar - x) * (xstar - x))) * fp(x).abs.inverse
        ((ddf(c).abs / taylor_two) * ((xstar - x) * (xstar - x))) * fp(x).abs.inverse =
            ((ddf(c).abs / taylor_two) * ((xstar - x) * (xstar - x))) / fp(x).abs
        ((bigm / taylor_two) * ((xstar - x) * (xstar - x))) * fp(x).abs.inverse =
            ((bigm / taylor_two) * ((xstar - x) * (xstar - x))) / fp(x).abs
        ((ddf(c).abs / taylor_two) * ((xstar - x) * (xstar - x))) / fp(x).abs <= ((bigm / taylor_two) * ((xstar - x) * (xstar - x))) / fp(x).abs
        (newton_step(f, fp, x) - xstar).abs <= ((bigm / taylor_two) * ((xstar - x) * (xstar - x))) / fp(x).abs
        // M is nonnegative.
        lte_refl(x)
        x <= x
        lt_imp_lte(x, xstar)
        x <= xstar
        forall(z: Real) { x <= z and z <= xstar implies ddf(z).abs <= bigm }
        x <= x and x <= xstar implies ddf(x).abs <= bigm
        ddf(x).abs <= bigm
        abs_gte_zero(ddf(x))
        ddf(x).abs >= Real.0
        lte_trans(Real.0, ddf(x).abs, bigm)
        Real.0 <= bigm
        // (bigm / taylor_two) * (xstar-x)^2 is nonnegative.
        lt_imp_lte(Real.0, taylor_two.inverse)
        Real.0 <= taylor_two.inverse
        taylor_two.inverse >= Real.0
        mul_nonneg(bigm, taylor_two.inverse)
        bigm * taylor_two.inverse >= Real.0
        Real.0 <= bigm * taylor_two.inverse
        bigm / taylor_two = bigm * taylor_two.inverse
        Real.0 <= bigm / taylor_two
        bigm / taylor_two >= Real.0
        (xstar - x) * (xstar - x) >= Real.0
        mul_nonneg(bigm / taylor_two, (xstar - x) * (xstar - x))
        (bigm / taylor_two) * ((xstar - x) * (xstar - x)) >= Real.0
        Real.0 <= (bigm / taylor_two) * ((xstar - x) * (xstar - x))
        // Reciprocals: 1/|fp(x)| <= 1/m, and multiplying by the nonnegative
        // factor (M/taylor_two) * (xstar-x)^2 gives the last inequality.
        pos_gt_zero(m)
        m > Real.0
        real_recip_antitone_pos(m, fp(x).abs)
        Real.1 / fp(x).abs <= Real.1 / m
        multiply_inequality_with_nonnegative_element[Real](Real.1 / fp(x).abs,
            Real.1 / m, (bigm / taylor_two) * ((xstar - x) * (xstar - x)))
        (Real.1 / fp(x).abs) * ((bigm / taylor_two) * ((xstar - x) * (xstar - x))) <= (Real.1 / m) * ((bigm / taylor_two) * ((xstar - x) * (xstar - x)))
        real_mul_comm(Real.1 / fp(x).abs,
            (bigm / taylor_two) * ((xstar - x) * (xstar - x)))
        (Real.1 / fp(x).abs) * ((bigm / taylor_two) * ((xstar - x) * (xstar - x))) =
            ((bigm / taylor_two) * ((xstar - x) * (xstar - x))) * (Real.1 / fp(x).abs)
        ((bigm / taylor_two) * ((xstar - x) * (xstar - x))) * (Real.1 / fp(x).abs) =
            ((bigm / taylor_two) * ((xstar - x) * (xstar - x))) / fp(x).abs
        ((bigm / taylor_two) * ((xstar - x) * (xstar - x))) / fp(x).abs <= (Real.1 / m) * ((bigm / taylor_two) * ((xstar - x) * (xstar - x)))
        real_mul_comm(Real.1 / m, (bigm / taylor_two) * ((xstar - x) * (xstar - x)))
        (Real.1 / m) * ((bigm / taylor_two) * ((xstar - x) * (xstar - x))) =
            ((bigm / taylor_two) * ((xstar - x) * (xstar - x))) * (Real.1 / m)
        ((bigm / taylor_two) * ((xstar - x) * (xstar - x))) * (Real.1 / m) =
            ((bigm / taylor_two) * ((xstar - x) * (xstar - x))) / m
        ((bigm / taylor_two) * ((xstar - x) * (xstar - x))) / fp(x).abs <= ((bigm / taylor_two) * ((xstar - x) * (xstar - x))) / m
        lte_trans((newton_step(f, fp, x) - xstar).abs,
            ((bigm / taylor_two) * ((xstar - x) * (xstar - x))) / fp(x).abs,
            ((bigm / taylor_two) * ((xstar - x) * (xstar - x))) / m)
        (newton_step(f, fp, x) - xstar).abs <= ((bigm / taylor_two) * ((xstar - x) * (xstar - x))) / m
        // Reassociate the product: (M/taylor_two) * (xstar-x) * (xstar-x).
        mul_assoc(bigm / taylor_two, xstar - x, xstar - x)
        ((bigm / taylor_two) * (xstar - x)) * (xstar - x) =
            (bigm / taylor_two) * ((xstar - x) * (xstar - x))
        (newton_step(f, fp, x) - xstar).abs <= ((bigm / taylor_two) * (xstar - x) * (xstar - x)) / m
    }
}
