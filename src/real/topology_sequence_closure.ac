from nat import Nat
from data.basic.set import Set
from real.real_field import Real
from real.real_seq import converges_to, tail_bound, tail_bound_implies_is_close
from real.sequence_set_membership import seq_eventually_in_real_set,
    seq_eventually_in_real_set_frequently, seq_frequently_in_real_set,
    seq_frequently_in_real_set_at, seq_in_real_set, seq_in_real_set_frequently
from real.topology import adherent_point_intro, closure, closure_contains_eq,
    closed_set_contains_adherent, eps_adherent_of_contains_close,
    is_adherent_point_of_set, is_closed_set, is_eps_adherent_to_set,
    is_limit_point_of_set
from real.topology_closure import closure_is_closed
from real.topology_sequences import converges_to_has_tail_bound

/// A tail bound and frequent membership give epsilon-adherence to the visited set.
theorem tail_bound_frequently_in_real_set_imp_eps_adherent(
    s: Set[Real], a: Nat -> Real, x: Real, eps: Real, n0: Nat
) {
    seq_frequently_in_real_set(s, a) and tail_bound(a, x, n0, eps)
    implies is_eps_adherent_to_set(s, x, eps)
} by {
    if seq_frequently_in_real_set(s, a) and tail_bound(a, x, n0, eps) {
        seq_frequently_in_real_set(s, a) = forall(n_start: Nat) {
            exists(n_witness: Nat) {
                n_start <= n_witness and s.contains(a(n_witness))
            }
        }
        seq_frequently_in_real_set_at(s, a, n0)
        exists(n_witness: Nat) {
            n0 <= n_witness and s.contains(a(n_witness))
        }
        let n_witness: Nat satisfy {
            n0 <= n_witness and s.contains(a(n_witness))
        }
        tail_bound_implies_is_close(a, x, n0, eps, n_witness)
        a(n_witness).is_close(x, eps)
        eps_adherent_of_contains_close(s, x, a(n_witness), eps)
        is_eps_adherent_to_set(s, x, eps)
    }
}

/// Frequent membership along a convergent sequence gives epsilon-adherence to the visited set.
theorem seq_frequently_converges_imp_eps_adherent(
    s: Set[Real], a: Nat -> Real, x: Real, eps: Real
) {
    seq_frequently_in_real_set(s, a) and converges_to(a, x) and eps.is_positive
    implies is_eps_adherent_to_set(s, x, eps)
} by {
    if seq_frequently_in_real_set(s, a) and converges_to(a, x) and eps.is_positive {
        converges_to_has_tail_bound(a, x, eps)
        let n0: Nat satisfy {
            tail_bound(a, x, n0, eps)
        }
        tail_bound_frequently_in_real_set_imp_eps_adherent(s, a, x, eps, n0)
        is_eps_adherent_to_set(s, x, eps)
    }
}

/// Frequent membership along a convergent sequence makes the limit adherent to the visited set.
theorem seq_frequently_converges_imp_adherent(s: Set[Real], a: Nat -> Real, x: Real) {
    seq_frequently_in_real_set(s, a) and converges_to(a, x)
    implies is_adherent_point_of_set(s, x)
} by {
    if seq_frequently_in_real_set(s, a) and converges_to(a, x) {
        forall(eps: Real) {
            if eps.is_positive {
                seq_frequently_converges_imp_eps_adherent(s, a, x, eps)
                is_eps_adherent_to_set(s, x, eps)
            }
        }
        adherent_point_intro(s, x)
        is_adherent_point_of_set(s, x)
    }
}

/// Frequent membership along a convergent sequence puts the limit in the closure of the visited set.
theorem seq_frequently_converges_imp_closure_contains(s: Set[Real], a: Nat -> Real, x: Real) {
    seq_frequently_in_real_set(s, a) and converges_to(a, x) implies closure(s).contains(x)
} by {
    if seq_frequently_in_real_set(s, a) and converges_to(a, x) {
        seq_frequently_converges_imp_adherent(s, a, x)
        is_adherent_point_of_set(s, x)
        closure_contains_eq(s, x)
        closure(s).contains(x)
    }
}

/// Eventual membership along a convergent sequence makes the limit adherent to the visited set.
theorem seq_eventually_converges_imp_adherent(s: Set[Real], a: Nat -> Real, x: Real) {
    seq_eventually_in_real_set(s, a) and converges_to(a, x)
    implies is_adherent_point_of_set(s, x)
} by {
    if seq_eventually_in_real_set(s, a) and converges_to(a, x) {
        seq_eventually_in_real_set_frequently(s, a)
        seq_frequently_in_real_set(s, a)
        seq_frequently_converges_imp_adherent(s, a, x)
        is_adherent_point_of_set(s, x)
    }
}

/// Eventual membership along a convergent sequence puts the limit in the closure of the visited set.
theorem seq_eventually_converges_imp_closure_contains(s: Set[Real], a: Nat -> Real, x: Real) {
    seq_eventually_in_real_set(s, a) and converges_to(a, x) implies closure(s).contains(x)
} by {
    if seq_eventually_in_real_set(s, a) and converges_to(a, x) {
        seq_eventually_converges_imp_adherent(s, a, x)
        is_adherent_point_of_set(s, x)
        closure_contains_eq(s, x)
        closure(s).contains(x)
    }
}

/// Sequence membership along a convergent sequence makes the limit adherent to the visited set.
theorem seq_in_real_set_converges_imp_adherent(s: Set[Real], a: Nat -> Real, x: Real) {
    seq_in_real_set(s, a) and converges_to(a, x) implies is_adherent_point_of_set(s, x)
} by {
    if seq_in_real_set(s, a) and converges_to(a, x) {
        seq_in_real_set_frequently(s, a)
        seq_frequently_in_real_set(s, a)
        seq_frequently_converges_imp_adherent(s, a, x)
        is_adherent_point_of_set(s, x)
    }
}

/// Sequence membership along a convergent sequence puts the limit in the closure of the visited set.
theorem seq_in_real_set_converges_imp_closure_contains(s: Set[Real], a: Nat -> Real, x: Real) {
    seq_in_real_set(s, a) and converges_to(a, x) implies closure(s).contains(x)
} by {
    if seq_in_real_set(s, a) and converges_to(a, x) {
        seq_in_real_set_converges_imp_adherent(s, a, x)
        is_adherent_point_of_set(s, x)
        closure_contains_eq(s, x)
        closure(s).contains(x)
    }
}

/// A closed set contains the limit of every frequently visiting convergent sequence.
theorem closed_set_contains_frequent_seq_limit(s: Set[Real], a: Nat -> Real, x: Real) {
    is_closed_set(s) and seq_frequently_in_real_set(s, a) and converges_to(a, x) implies s.contains(x)
} by {
    if is_closed_set(s) and seq_frequently_in_real_set(s, a) and converges_to(a, x) {
        seq_frequently_converges_imp_adherent(s, a, x)
        is_adherent_point_of_set(s, x)
        closed_set_contains_adherent(s, x)
        s.contains(x)
    }
}

/// A closed set contains the limit of every eventually contained convergent sequence.
theorem closed_set_contains_eventual_seq_limit(s: Set[Real], a: Nat -> Real, x: Real) {
    is_closed_set(s) and seq_eventually_in_real_set(s, a) and converges_to(a, x) implies s.contains(x)
} by {
    if is_closed_set(s) and seq_eventually_in_real_set(s, a) and converges_to(a, x) {
        seq_eventually_in_real_set_frequently(s, a)
        seq_frequently_in_real_set(s, a)
        closed_set_contains_frequent_seq_limit(s, a, x)
        s.contains(x)
    }
}

/// A closed set contains the limit of every contained convergent sequence.
theorem closed_set_contains_seq_in_real_set_limit(s: Set[Real], a: Nat -> Real, x: Real) {
    is_closed_set(s) and seq_in_real_set(s, a) and converges_to(a, x) implies s.contains(x)
} by {
    if is_closed_set(s) and seq_in_real_set(s, a) and converges_to(a, x) {
        seq_in_real_set_frequently(s, a)
        seq_frequently_in_real_set(s, a)
        closed_set_contains_frequent_seq_limit(s, a, x)
        s.contains(x)
    }
}

/// A convergent sequence frequently in a closure has its limit in that closure.
theorem seq_frequently_in_closure_converges_imp_closure_contains(
    s: Set[Real], a: Nat -> Real, x: Real
) {
    seq_frequently_in_real_set(closure(s), a) and converges_to(a, x) implies closure(s).contains(x)
} by {
    if seq_frequently_in_real_set(closure(s), a) and converges_to(a, x) {
        closure_is_closed(s)
        is_closed_set(closure(s))
        closed_set_contains_frequent_seq_limit(closure(s), a, x)
        closure(s).contains(x)
    }
}

/// A convergent sequence eventually in a closure has its limit in that closure.
theorem seq_eventually_in_closure_converges_imp_closure_contains(
    s: Set[Real], a: Nat -> Real, x: Real
) {
    seq_eventually_in_real_set(closure(s), a) and converges_to(a, x) implies closure(s).contains(x)
} by {
    if seq_eventually_in_real_set(closure(s), a) and converges_to(a, x) {
        closure_is_closed(s)
        is_closed_set(closure(s))
        closed_set_contains_eventual_seq_limit(closure(s), a, x)
        closure(s).contains(x)
    }
}

/// A convergent sequence contained in a closure has its limit in that closure.
theorem seq_in_closure_converges_imp_closure_contains(s: Set[Real], a: Nat -> Real, x: Real) {
    seq_in_real_set(closure(s), a) and converges_to(a, x) implies closure(s).contains(x)
} by {
    if seq_in_real_set(closure(s), a) and converges_to(a, x) {
        closure_is_closed(s)
        is_closed_set(closure(s))
        closed_set_contains_seq_in_real_set_limit(closure(s), a, x)
        closure(s).contains(x)
    }
}

/// Frequent visits to a punctured set by a sequence converging to the puncture give a limit point.
theorem seq_frequently_in_punctured_set_converges_imp_limit_point(
    s: Set[Real], a: Nat -> Real, x: Real
) {
    seq_frequently_in_real_set(s.difference(Set[Real].singleton(x)), a) and converges_to(a, x)
    implies is_limit_point_of_set(s, x)
} by {
    if seq_frequently_in_real_set(s.difference(Set[Real].singleton(x)), a) and converges_to(a, x) {
        seq_frequently_converges_imp_adherent(s.difference(Set[Real].singleton(x)), a, x)
        is_adherent_point_of_set(s.difference(Set[Real].singleton(x)), x)
        is_limit_point_of_set(s, x)
    }
}

/// Eventual visits to a punctured set by a sequence converging to the puncture give a limit point.
theorem seq_eventually_in_punctured_set_converges_imp_limit_point(
    s: Set[Real], a: Nat -> Real, x: Real
) {
    seq_eventually_in_real_set(s.difference(Set[Real].singleton(x)), a) and converges_to(a, x)
    implies is_limit_point_of_set(s, x)
} by {
    if seq_eventually_in_real_set(s.difference(Set[Real].singleton(x)), a) and converges_to(a, x) {
        seq_eventually_in_real_set_frequently(s.difference(Set[Real].singleton(x)), a)
        seq_frequently_in_real_set(s.difference(Set[Real].singleton(x)), a)
        seq_frequently_in_punctured_set_converges_imp_limit_point(s, a, x)
        is_limit_point_of_set(s, x)
    }
}

/// A contained sequence converging to a point outside a set is frequently in the punctured set.
theorem seq_in_real_set_converges_to_external_point_imp_frequently_punctured(
    s: Set[Real], a: Nat -> Real, x: Real
) {
    seq_in_real_set(s, a) and not s.contains(x)
    implies seq_frequently_in_real_set(s.difference(Set[Real].singleton(x)), a)
} by {
    if seq_in_real_set(s, a) and not s.contains(x) {
        forall(n_start: Nat) {
            seq_in_real_set(s, a) = forall(n: Nat) {
                s.contains(a(n))
            }
            s.contains(a(n_start))
            if Set[Real].singleton(x).contains(a(n_start)) {
                a(n_start) = x
                s.contains(x)
                false
            }
            not Set[Real].singleton(x).contains(a(n_start))
            s.difference(Set[Real].singleton(x)).contains(a(n_start))
            exists(n_witness: Nat) {
                n_start <= n_witness and s.difference(Set[Real].singleton(x)).contains(a(n_witness))
            }
        }
        seq_frequently_in_real_set(s.difference(Set[Real].singleton(x)), a)
    }
}

/// A convergent sequence contained in a set but converging outside it gives a limit point of the set.
theorem seq_in_real_set_converges_to_external_point_imp_limit_point(
    s: Set[Real], a: Nat -> Real, x: Real
) {
    seq_in_real_set(s, a) and converges_to(a, x) and not s.contains(x) implies is_limit_point_of_set(s, x)
} by {
    if seq_in_real_set(s, a) and converges_to(a, x) and not s.contains(x) {
        seq_in_real_set_converges_to_external_point_imp_frequently_punctured(s, a, x)
        seq_frequently_in_real_set(s.difference(Set[Real].singleton(x)), a)
        seq_frequently_in_punctured_set_converges_imp_limit_point(s, a, x)
        is_limit_point_of_set(s, x)
    }
}
