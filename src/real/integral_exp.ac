/// The integral of the exponential function over a closed interval.
///
/// The main result, integral_exp, states that the Darboux integral of Real.exp
/// over [a, b] equals b.exp - a.exp: Real.exp is its own integral up to the
/// endpoints.  The route is the fundamental theorem of calculus:
///   - Real.exp is continuous (exp_continuous, from differentiability via
///     derivative_continuous_at),
///   - Real.exp is its own derivative (exp_is_derivative_fn, from
///     derivative_exp_log.ac),
///   - the mean value theorem applied on each subinterval of a partition
///     sandwiches the Darboux sums of Real.exp between the increments
///     (p(i+1)).exp - (p(i)).exp, which telescope to b.exp - a.exp
///     (lower_sum_le_g_diff and g_diff_le_upper_sum, proved generically for
///     any g with g' = f in ftc2_general),
///   - integrability of Real.exp is established by comparing upper and lower
///     Darboux sums over the uniform partitions of [a, b], whose difference
///     is (b - a) * (b.exp - a.exp) / (n + 1) and so can be made
///     arbitrarily small (exp_integrable).

from nat import Nat, from_nat, from_nat_add, lt_imp_lte_suc
from rat import Rat
from order import lte_antisymm, lte_trans, lt_imp_lte, lt_imp_ne, lt_imp_ne_symm, not_lt_imp_gte, lt_of_lt_of_lte, lt_of_lte_of_lt, not_lte_imp_gt, not_lt_self
from real.real_field import Real, mul_div, mul_inverse
from real.exp import exp_zero, exp_increasing, exp_pos, exists_nat_gt, inverse_pos
from real.derivative_exp_log import exp_has_derivative_at, exp_is_derivative_fn
from real.derivative_basic import has_derivative_at, has_derivative_at_unique, sub_ne_zero_of_ne
from real.derivative_continuity import derivative_continuous_at, div_mul_cancel_denominator
from real.calculus_api import is_derivative_fn, is_derivative_fn_at
from real.continuity_base import continuous, continuous_at
from real.mean_value import mean_value_theorem, secant_slope
from real.integral import integral, is_integrable, interval_contains, interval_set, interval_image, interval_inf, interval_sup, interval_inf_spec, interval_sup_spec, lower_sum, upper_sum, lower_sum_set, upper_sum_set, lower_sum_contains, upper_sum_contains, partition_step_lower, partition_step_upper, is_partition, partition_start, partition_end, partition_mono, partition_point_in_interval, partition_monotone, diff_step, telescope, partial_lte, image_lower_bound, interval_set_contains_left, interval_set_contains_right, interval_contains_left, interval_contains_right, interval_contains_mono, sub_nonneg, integral_spec, set_supremum_unique, set_infimum_unique, sup_le_of_upper_bound, trivial_partition, trivial_partition_is_partition, neg_lte_flip, set_infimum_is_lower_bound, set_lower_bound_contains_le, set_lower_bound_le_infimum, has_lower_bound
from real.supremum import completeness, is_nonempty, has_upper_bound, is_set_supremum, is_set_upper_bound, is_set_infimum, is_set_lower_bound, set_member_le_supremum, set_supremum_le_upper_bound, set_upper_bound_contains_le, function_image, function_image_contains, negate_set, negate_set_contains
from real.integral import neg_upper_bound_of_lower, negate_set_nonempty, inf_of_neg_sup
from real.double_sum import partial_sub_seq, sub_seq
from list import partial, partial_pointwise_eq, partial_scalar_mul
from algebra.semigroup import mul_fn
from algebra.add_ordered_group import add_le_add, add_le_add_right
from ordered_field import mul_le_mul_of_nonneg_right
from data.basic.set import Set, maps_into_set_image
from real.harmonic import from_nat_suc_pos_real, rat_from_nat_lte_of_nat_lte
from real.real_base import from_rat_maintains_lte, gt_zero_imp_pos, pos_gt_zero, lte_lt_trans, add_comm, add_assoc, neg_distrib
from real.real_ring import from_nat_is_from_rat, mul_le_mul_nonneg, mul_pos_pos, lt_mul_pos_left, lte_mul_nonneg_right, mul_sub_distrib_left
from real.real_seq import sub_zero_imp_eq

numerals Real
numerals Nat

// ---------------------------------------------------------------------------
// The exponential is continuous and monotone
// ---------------------------------------------------------------------------

/// The exponential function is continuous at every point.
theorem exp_continuous {
    continuous(Real.exp)
} by {
    forall(x: Real) {
        exp_has_derivative_at(x)
        has_derivative_at(Real.exp, x, x.exp)
        derivative_continuous_at(Real.exp, x, x.exp)
        continuous_at(Real.exp, x)
    }
    continuous(Real.exp) = forall(x: Real) {
        continuous_at(Real.exp, x)
    }
    continuous(Real.exp)
}

/// The exponential function is nondecreasing: x <= y implies x.exp <= y.exp.
theorem exp_lte_mono(x: Real, y: Real) {
    x <= y implies x.exp <= y.exp
} by {
    if x <= y {
        if x < y {
            exp_increasing(x, y)
            x.exp < y.exp
            lt_imp_lte(x.exp, y.exp)
            x.exp <= y.exp
        } else {
            not x < y
            not_lt_imp_gte[Real](x, y)
            y <= x
            lte_antisymm[Real](x, y)
            x = y
            x.exp = y.exp
            x.exp <= y.exp
        }
    }
}

/// On [a, b] the exponential is bounded below by a.exp.
theorem exp_lower_bound_on(a: Real, b: Real) {
    a <= b implies forall(t: Real) { interval_contains(a, b, t) implies a.exp <= t.exp }
} by {
    if a <= b {
        forall(t: Real) {
            if interval_contains(a, b, t) {
                interval_contains_left(a, b, t)
                a <= t
                exp_lte_mono(a, t)
                a.exp <= t.exp
            }
        }
    }
}

/// On [a, b] the exponential is bounded above by b.exp.
theorem exp_upper_bound_on(a: Real, b: Real) {
    a <= b implies forall(t: Real) { interval_contains(a, b, t) implies t.exp <= b.exp }
} by {
    if a <= b {
        forall(t: Real) {
            if interval_contains(a, b, t) {
                interval_contains_right(a, b, t)
                t <= b
                exp_lte_mono(t, b)
                t.exp <= b.exp
            }
        }
    }
}

/// A pointwise upper bound of f over [x, y] makes it an upper bound of the image.
theorem image_upper_bound(f: Real -> Real, x: Real, y: Real, ub: Real) {
    (forall(t: Real) { interval_contains(x, y, t) implies f(t) <= ub })
    implies
    is_set_upper_bound(interval_image(f, x, y), ub)
} by {
    if forall(t: Real) { interval_contains(x, y, t) implies f(t) <= ub } {
        forall(v: Real) {
            if interval_image(f, x, y).contains(v) {
                interval_image(f, x, y).contains(v) = function_image_contains(f, interval_set(x, y), v)
                function_image_contains(f, interval_set(x, y), v)
                let t: Real satisfy {
                    interval_set(x, y).contains(t) and v = f(t)
                }
                interval_set(x, y).contains(t) = interval_contains(x, y, t)
                interval_contains(x, y, t)
                forall(t0: Real) {
                    interval_contains(x, y, t0) implies f(t0) <= ub
                }
                interval_contains(x, y, t) implies f(t) <= ub
                f(t) <= ub
                v = f(t)
                v <= ub
            }
        }
        is_set_upper_bound(interval_image(f, x, y), ub)
    }
}

// ---------------------------------------------------------------------------
// The fundamental theorem of calculus, part 2 (special case)
// ---------------------------------------------------------------------------
//
// If g is continuous with pointwise derivative f, f is bounded on [a, b] and
// integrable, then the integral of f over [a, b] is g(b) - g(a).  The proof
// applies the mean value theorem on each subinterval of a partition: on
// [p(i), p(i+1)] there is c with g(p(i+1)) - g(p(i)) = f(c) * (p(i+1) - p(i)),
// so the increment of g across the subinterval lies between the lower and
// upper Darboux steps of f there; summing over the partition and telescoping
// sandwiches every lower sum of f below g(b) - g(a) and every upper sum
// above it, which forces the integral (the common supremum of lower sums and
// infimum of upper sums) to equal g(b) - g(a).

/// The values of g at the points of a partition.
define partition_g_values(g: Real -> Real, p: Nat -> Real, i: Nat) -> Real {
    g(p(i))
}

/// The lower step of f over a subinterval of a partition of [a, b] is at most
/// the increment of g across that subinterval, when g has derivative f and f
/// is bounded below on [a, b].
theorem step_lower_le_g_increment(f: Real -> Real, g: Real -> Real, a: Real, b: Real, lb: Real, p: Nat -> Real, n: Nat, i: Nat) {
    is_partition(p, a, b, n) and i < n and a <= b and continuous(g) and is_derivative_fn(g, f) and
    (forall(t: Real) { interval_contains(a, b, t) implies lb <= f(t) })
    implies
    partition_step_lower(f, p, i) <= diff_step(partition_g_values(g, p), i)
} by {
    if is_partition(p, a, b, n) and i < n and a <= b and continuous(g) and is_derivative_fn(g, f) and
       (forall(t: Real) { interval_contains(a, b, t) implies lb <= f(t) }) {
        lt_imp_lte_suc(i, n)
        i + 1 <= n
        i <= i + 1
        partition_mono(p, a, b, n, i, i + 1)
        p(i) <= p(i + 1)
        if p(i) < p(i + 1) {
            mean_value_theorem(g, f, p(i), p(i + 1))
            let c: Real satisfy {
                p(i) < c and c < p(i + 1) and has_derivative_at(g, c, secant_slope(g, p(i), p(i + 1)))
            }
            p(i) < c and c < p(i + 1)
            has_derivative_at(g, c, secant_slope(g, p(i), p(i + 1)))
            is_derivative_fn_at(g, f, c)
            has_derivative_at(g, c, f(c))
            has_derivative_at_unique(g, c, secant_slope(g, p(i), p(i + 1)), f(c))
            secant_slope(g, p(i), p(i + 1)) = f(c)
            secant_slope(g, p(i), p(i + 1)) = (g(p(i + 1)) - g(p(i))) / (p(i + 1) - p(i))
            f(c) = (g(p(i + 1)) - g(p(i))) / (p(i + 1) - p(i))
            lt_imp_ne_symm(p(i), p(i + 1))
            p(i + 1) != p(i)
            sub_ne_zero_of_ne(p(i + 1), p(i))
            p(i + 1) - p(i) != Real.0
            div_mul_cancel_denominator(g(p(i + 1)) - g(p(i)), p(i + 1) - p(i))
            ((g(p(i + 1)) - g(p(i))) / (p(i + 1) - p(i))) * (p(i + 1) - p(i)) = g(p(i + 1)) - g(p(i))
            f(c) * (p(i + 1) - p(i)) = g(p(i + 1)) - g(p(i))
            // c lies in the closed subinterval, so f(c) is a member of the image.
            lt_imp_lte(p(i), c)
            p(i) <= c
            lt_imp_lte(c, p(i + 1))
            c <= p(i + 1)
            interval_contains(p(i), p(i + 1), c)
            interval_set(p(i), p(i + 1)).contains(c) = interval_contains(p(i), p(i + 1), c)
            interval_set(p(i), p(i + 1)).contains(c)
            maps_into_set_image(interval_set(p(i), p(i + 1)), f, c)
            function_image(f, interval_set(p(i), p(i + 1))).contains(f(c))
            interval_image(f, p(i), p(i + 1)).contains(f(c))
            // f is bounded below on the subinterval by lb, since the
            // subinterval lies inside [a, b].
            forall(t: Real) {
                if interval_contains(p(i), p(i + 1), t) {
                    interval_contains_left(p(i), p(i + 1), t)
                    p(i) <= t
                    interval_contains_right(p(i), p(i + 1), t)
                    t <= p(i + 1)
                    partition_point_in_interval(p, a, b, n, i)
                    interval_contains(a, b, p(i))
                    partition_point_in_interval(p, a, b, n, i + 1)
                    interval_contains(a, b, p(i + 1))
                    interval_contains_mono(a, b, p(i), p(i + 1), t)
                    interval_contains(a, b, t)
                    forall(t0: Real) {
                        interval_contains(a, b, t0) implies lb <= f(t0)
                    }
                    interval_contains(a, b, t) implies lb <= f(t)
                    lb <= f(t)
                }
            }
            image_lower_bound(f, p(i), p(i + 1), lb)
            is_set_lower_bound(interval_image(f, p(i), p(i + 1)), lb)
            exists(b0: Real) {
                is_set_lower_bound(interval_image(f, p(i), p(i + 1)), b0)
            }
            has_lower_bound(interval_image(f, p(i), p(i + 1)))
            interval_set_contains_left(p(i), p(i + 1))
            is_nonempty(interval_set(p(i), p(i + 1)))
            is_nonempty(interval_set(p(i), p(i + 1))) and has_lower_bound(interval_image(f, p(i), p(i + 1)))
            interval_inf_spec(f, p(i), p(i + 1))
            is_set_infimum(interval_image(f, p(i), p(i + 1)), interval_inf(f, p(i), p(i + 1)))
            set_infimum_is_lower_bound(interval_image(f, p(i), p(i + 1)), interval_inf(f, p(i), p(i + 1)))
            is_set_lower_bound(interval_image(f, p(i), p(i + 1)), interval_inf(f, p(i), p(i + 1)))
            set_lower_bound_contains_le(interval_image(f, p(i), p(i + 1)), interval_inf(f, p(i), p(i + 1)), f(c))
            interval_inf(f, p(i), p(i + 1)) <= f(c)
            // The width is nonnegative, so multiplying preserves the order.
            sub_nonneg(p(i), p(i + 1))
            Real.0 <= p(i + 1) - p(i)
            mul_le_mul_of_nonneg_right(interval_inf(f, p(i), p(i + 1)), f(c), p(i + 1) - p(i))
            interval_inf(f, p(i), p(i + 1)) * (p(i + 1) - p(i)) <= f(c) * (p(i + 1) - p(i))
            partition_step_lower(f, p, i) = interval_inf(f, p(i), p(i + 1)) * diff_step(p, i)
            diff_step(p, i) = p(i + 1) - p(i)
            partition_step_lower(f, p, i) = interval_inf(f, p(i), p(i + 1)) * (p(i + 1) - p(i))
            partition_step_lower(f, p, i) <= f(c) * (p(i + 1) - p(i))
            partition_step_lower(f, p, i) <= g(p(i + 1)) - g(p(i))
            diff_step(partition_g_values(g, p), i) = partition_g_values(g, p, i + 1) - partition_g_values(g, p, i)
            partition_g_values(g, p, i + 1) = g(p(i + 1))
            partition_g_values(g, p, i) = g(p(i))
            diff_step(partition_g_values(g, p), i) = g(p(i + 1)) - g(p(i))
            partition_step_lower(f, p, i) <= diff_step(partition_g_values(g, p), i)
        } else {
            // The subinterval is degenerate: both steps vanish.
            not p(i) < p(i + 1)
            not_lt_imp_gte[Real](p(i), p(i + 1))
            p(i + 1) <= p(i)
            lte_antisymm[Real](p(i), p(i + 1))
            p(i) = p(i + 1)
            partition_step_lower(f, p, i) = interval_inf(f, p(i), p(i + 1)) * diff_step(p, i)
            diff_step(p, i) = p(i + 1) - p(i)
            partition_step_lower(f, p, i) = interval_inf(f, p(i), p(i + 1)) * (p(i + 1) - p(i))
            p(i + 1) - p(i) = Real.0
            partition_step_lower(f, p, i) = interval_inf(f, p(i), p(i + 1)) * Real.0
            interval_inf(f, p(i), p(i + 1)) * Real.0 = Real.0
            partition_step_lower(f, p, i) = Real.0
            diff_step(partition_g_values(g, p), i) = partition_g_values(g, p, i + 1) - partition_g_values(g, p, i)
            partition_g_values(g, p, i + 1) = g(p(i + 1))
            partition_g_values(g, p, i) = g(p(i))
            diff_step(partition_g_values(g, p), i) = g(p(i + 1)) - g(p(i))
            g(p(i + 1)) = g(p(i))
            g(p(i + 1)) - g(p(i)) = Real.0
            diff_step(partition_g_values(g, p), i) = Real.0
            partition_step_lower(f, p, i) <= diff_step(partition_g_values(g, p), i)
        }
    }
}

/// The increment of g across a subinterval of a partition of [a, b] is at
/// most the upper step of f there, when g has derivative f and f is bounded
/// above on [a, b].
theorem step_upper_ge_g_increment(f: Real -> Real, g: Real -> Real, a: Real, b: Real, ub: Real, p: Nat -> Real, n: Nat, i: Nat) {
    is_partition(p, a, b, n) and i < n and a <= b and continuous(g) and is_derivative_fn(g, f) and
    (forall(t: Real) { interval_contains(a, b, t) implies f(t) <= ub })
    implies
    diff_step(partition_g_values(g, p), i) <= partition_step_upper(f, p, i)
} by {
    if is_partition(p, a, b, n) and i < n and a <= b and continuous(g) and is_derivative_fn(g, f) and
       (forall(t: Real) { interval_contains(a, b, t) implies f(t) <= ub }) {
        lt_imp_lte_suc(i, n)
        i + 1 <= n
        i <= i + 1
        partition_mono(p, a, b, n, i, i + 1)
        p(i) <= p(i + 1)
        if p(i) < p(i + 1) {
            mean_value_theorem(g, f, p(i), p(i + 1))
            let c: Real satisfy {
                p(i) < c and c < p(i + 1) and has_derivative_at(g, c, secant_slope(g, p(i), p(i + 1)))
            }
            p(i) < c and c < p(i + 1)
            has_derivative_at(g, c, secant_slope(g, p(i), p(i + 1)))
            is_derivative_fn_at(g, f, c)
            has_derivative_at(g, c, f(c))
            has_derivative_at_unique(g, c, secant_slope(g, p(i), p(i + 1)), f(c))
            secant_slope(g, p(i), p(i + 1)) = f(c)
            secant_slope(g, p(i), p(i + 1)) = (g(p(i + 1)) - g(p(i))) / (p(i + 1) - p(i))
            f(c) = (g(p(i + 1)) - g(p(i))) / (p(i + 1) - p(i))
            lt_imp_ne_symm(p(i), p(i + 1))
            p(i + 1) != p(i)
            sub_ne_zero_of_ne(p(i + 1), p(i))
            p(i + 1) - p(i) != Real.0
            div_mul_cancel_denominator(g(p(i + 1)) - g(p(i)), p(i + 1) - p(i))
            ((g(p(i + 1)) - g(p(i))) / (p(i + 1) - p(i))) * (p(i + 1) - p(i)) = g(p(i + 1)) - g(p(i))
            f(c) * (p(i + 1) - p(i)) = g(p(i + 1)) - g(p(i))
            lt_imp_lte(p(i), c)
            p(i) <= c
            lt_imp_lte(c, p(i + 1))
            c <= p(i + 1)
            interval_contains(p(i), p(i + 1), c)
            interval_set(p(i), p(i + 1)).contains(c) = interval_contains(p(i), p(i + 1), c)
            interval_set(p(i), p(i + 1)).contains(c)
            maps_into_set_image(interval_set(p(i), p(i + 1)), f, c)
            function_image(f, interval_set(p(i), p(i + 1))).contains(f(c))
            interval_image(f, p(i), p(i + 1)).contains(f(c))
            forall(t: Real) {
                if interval_contains(p(i), p(i + 1), t) {
                    interval_contains_left(p(i), p(i + 1), t)
                    p(i) <= t
                    interval_contains_right(p(i), p(i + 1), t)
                    t <= p(i + 1)
                    partition_point_in_interval(p, a, b, n, i)
                    interval_contains(a, b, p(i))
                    partition_point_in_interval(p, a, b, n, i + 1)
                    interval_contains(a, b, p(i + 1))
                    interval_contains_mono(a, b, p(i), p(i + 1), t)
                    interval_contains(a, b, t)
                    forall(t0: Real) {
                        interval_contains(a, b, t0) implies f(t0) <= ub
                    }
                    interval_contains(a, b, t) implies f(t) <= ub
                    f(t) <= ub
                }
            }
            image_upper_bound(f, p(i), p(i + 1), ub)
            is_set_upper_bound(interval_image(f, p(i), p(i + 1)), ub)
            exists(b0: Real) {
                is_set_upper_bound(interval_image(f, p(i), p(i + 1)), b0)
            }
            has_upper_bound(interval_image(f, p(i), p(i + 1)))
            interval_set_contains_left(p(i), p(i + 1))
            is_nonempty(interval_set(p(i), p(i + 1)))
            is_nonempty(interval_set(p(i), p(i + 1))) and has_upper_bound(interval_image(f, p(i), p(i + 1)))
            interval_sup_spec(f, p(i), p(i + 1))
            is_set_supremum(interval_image(f, p(i), p(i + 1)), interval_sup(f, p(i), p(i + 1)))
            set_member_le_supremum(interval_image(f, p(i), p(i + 1)), interval_sup(f, p(i), p(i + 1)), f(c))
            f(c) <= interval_sup(f, p(i), p(i + 1))
            sub_nonneg(p(i), p(i + 1))
            Real.0 <= p(i + 1) - p(i)
            mul_le_mul_of_nonneg_right(f(c), interval_sup(f, p(i), p(i + 1)), p(i + 1) - p(i))
            f(c) * (p(i + 1) - p(i)) <= interval_sup(f, p(i), p(i + 1)) * (p(i + 1) - p(i))
            g(p(i + 1)) - g(p(i)) <= interval_sup(f, p(i), p(i + 1)) * (p(i + 1) - p(i))
            partition_step_upper(f, p, i) = interval_sup(f, p(i), p(i + 1)) * diff_step(p, i)
            diff_step(p, i) = p(i + 1) - p(i)
            partition_step_upper(f, p, i) = interval_sup(f, p(i), p(i + 1)) * (p(i + 1) - p(i))
            g(p(i + 1)) - g(p(i)) <= partition_step_upper(f, p, i)
            diff_step(partition_g_values(g, p), i) = partition_g_values(g, p, i + 1) - partition_g_values(g, p, i)
            partition_g_values(g, p, i + 1) = g(p(i + 1))
            partition_g_values(g, p, i) = g(p(i))
            diff_step(partition_g_values(g, p), i) = g(p(i + 1)) - g(p(i))
            diff_step(partition_g_values(g, p), i) <= partition_step_upper(f, p, i)
        } else {
            not p(i) < p(i + 1)
            not_lt_imp_gte[Real](p(i), p(i + 1))
            p(i + 1) <= p(i)
            lte_antisymm[Real](p(i), p(i + 1))
            p(i) = p(i + 1)
            diff_step(partition_g_values(g, p), i) = partition_g_values(g, p, i + 1) - partition_g_values(g, p, i)
            partition_g_values(g, p, i + 1) = g(p(i + 1))
            partition_g_values(g, p, i) = g(p(i))
            diff_step(partition_g_values(g, p), i) = g(p(i + 1)) - g(p(i))
            g(p(i + 1)) = g(p(i))
            g(p(i + 1)) - g(p(i)) = Real.0
            diff_step(partition_g_values(g, p), i) = Real.0
            partition_step_upper(f, p, i) = interval_sup(f, p(i), p(i + 1)) * diff_step(p, i)
            diff_step(p, i) = p(i + 1) - p(i)
            partition_step_upper(f, p, i) = interval_sup(f, p(i), p(i + 1)) * (p(i + 1) - p(i))
            p(i + 1) - p(i) = Real.0
            partition_step_upper(f, p, i) = interval_sup(f, p(i), p(i + 1)) * Real.0
            interval_sup(f, p(i), p(i + 1)) * Real.0 = Real.0
            partition_step_upper(f, p, i) = Real.0
            diff_step(partition_g_values(g, p), i) <= partition_step_upper(f, p, i)
        }
    }
}

/// Every lower Darboux sum of f over a partition of [a, b] is at most
/// g(b) - g(a), when g has derivative f and f is bounded below on [a, b].
theorem lower_sum_le_g_diff(f: Real -> Real, g: Real -> Real, a: Real, b: Real, lb: Real, p: Nat -> Real, n: Nat) {
    a <= b and continuous(g) and is_derivative_fn(g, f) and
    is_partition(p, a, b, n) and
    (forall(t: Real) { interval_contains(a, b, t) implies lb <= f(t) })
    implies
    lower_sum(f, p, n) <= g(b) - g(a)
} by {
    if a <= b and continuous(g) and is_derivative_fn(g, f) and
       is_partition(p, a, b, n) and
       (forall(t: Real) { interval_contains(a, b, t) implies lb <= f(t) }) {
        forall(k: Nat) {
            if k < n {
                step_lower_le_g_increment(f, g, a, b, lb, p, n, k)
                partition_step_lower(f, p, k) <= diff_step(partition_g_values(g, p), k)
            }
        }
        partial_lte(partition_step_lower(f, p), diff_step(partition_g_values(g, p)), n)
        partial(partition_step_lower(f, p), n) <= partial(diff_step(partition_g_values(g, p)), n)
        telescope(partition_g_values(g, p), n)
        partial(diff_step(partition_g_values(g, p)), n) = partition_g_values(g, p, n) - partition_g_values(g, p, Nat.0)
        partition_end(p, a, b, n)
        p(n) = b
        partition_start(p, a, b, n)
        p(Nat.0) = a
        partition_g_values(g, p, n) = g(p(n))
        partition_g_values(g, p, n) = g(b)
        partition_g_values(g, p, Nat.0) = g(p(Nat.0))
        partition_g_values(g, p, Nat.0) = g(a)
        partition_g_values(g, p, n) - partition_g_values(g, p, Nat.0) = g(b) - g(a)
        partial(diff_step(partition_g_values(g, p)), n) = g(b) - g(a)
        partial(partition_step_lower(f, p), n) <= g(b) - g(a)
        lower_sum(f, p, n) = partial(partition_step_lower(f, p), n)
        lower_sum(f, p, n) <= g(b) - g(a)
    }
}

/// Every upper Darboux sum of f over a partition of [a, b] is at least
/// g(b) - g(a), when g has derivative f and f is bounded above on [a, b].
theorem g_diff_le_upper_sum(f: Real -> Real, g: Real -> Real, a: Real, b: Real, ub: Real, p: Nat -> Real, n: Nat) {
    a <= b and continuous(g) and is_derivative_fn(g, f) and
    is_partition(p, a, b, n) and
    (forall(t: Real) { interval_contains(a, b, t) implies f(t) <= ub })
    implies
    g(b) - g(a) <= upper_sum(f, p, n)
} by {
    if a <= b and continuous(g) and is_derivative_fn(g, f) and
       is_partition(p, a, b, n) and
       (forall(t: Real) { interval_contains(a, b, t) implies f(t) <= ub }) {
        forall(k: Nat) {
            if k < n {
                step_upper_ge_g_increment(f, g, a, b, ub, p, n, k)
                diff_step(partition_g_values(g, p), k) <= partition_step_upper(f, p, k)
            }
        }
        partial_lte(diff_step(partition_g_values(g, p)), partition_step_upper(f, p), n)
        partial(diff_step(partition_g_values(g, p)), n) <= partial(partition_step_upper(f, p), n)
        telescope(partition_g_values(g, p), n)
        partial(diff_step(partition_g_values(g, p)), n) = partition_g_values(g, p, n) - partition_g_values(g, p, Nat.0)
        partition_end(p, a, b, n)
        p(n) = b
        partition_start(p, a, b, n)
        p(Nat.0) = a
        partition_g_values(g, p, n) = g(p(n))
        partition_g_values(g, p, n) = g(b)
        partition_g_values(g, p, Nat.0) = g(p(Nat.0))
        partition_g_values(g, p, Nat.0) = g(a)
        partition_g_values(g, p, n) - partition_g_values(g, p, Nat.0) = g(b) - g(a)
        partial(diff_step(partition_g_values(g, p)), n) = g(b) - g(a)
        g(b) - g(a) <= partial(partition_step_upper(f, p), n)
        upper_sum(f, p, n) = partial(partition_step_upper(f, p), n)
        g(b) - g(a) <= upper_sum(f, p, n)
    }
}

/// The fundamental theorem of calculus, part 2: if g is continuous with
/// pointwise derivative f, f is bounded on [a, b] and integrable there, then
/// the integral of f over [a, b] is g(b) - g(a).
theorem ftc2_general(f: Real -> Real, g: Real -> Real, a: Real, b: Real, lb: Real, ub: Real) {
    a <= b and continuous(g) and is_derivative_fn(g, f) and is_integrable(f, a, b) and
    (forall(t: Real) { interval_contains(a, b, t) implies lb <= f(t) }) and
    (forall(t: Real) { interval_contains(a, b, t) implies f(t) <= ub })
    implies
    integral(f, a, b) = g(b) - g(a)
} by {
    if a <= b and continuous(g) and is_derivative_fn(g, f) and is_integrable(f, a, b) and
       (forall(t: Real) { interval_contains(a, b, t) implies lb <= f(t) }) and
       (forall(t: Real) { interval_contains(a, b, t) implies f(t) <= ub }) {
        // The integral is the supremum of the lower sums and the infimum of
        // the upper sums.
        integral_spec(f, a, b)
        is_set_supremum(lower_sum_set(f, a, b), integral(f, a, b)) and is_set_infimum(upper_sum_set(f, a, b), integral(f, a, b))
        is_set_supremum(lower_sum_set(f, a, b), integral(f, a, b))
        is_set_infimum(upper_sum_set(f, a, b), integral(f, a, b))
        // Every lower sum is at most g(b) - g(a), so g(b) - g(a) is an upper
        // bound of the lower sums and the supremum lies below it.
        forall(s: Real) {
            if lower_sum_set(f, a, b).contains(s) {
                lower_sum_set(f, a, b).contains(s) = lower_sum_contains(f, a, b, s)
                lower_sum_contains(f, a, b, s)
                let (p: Nat -> Real, n: Nat) satisfy {
                    is_partition(p, a, b, n) and s = lower_sum(f, p, n)
                }
                is_partition(p, a, b, n)
                lower_sum_le_g_diff(f, g, a, b, lb, p, n)
                lower_sum(f, p, n) <= g(b) - g(a)
                s = lower_sum(f, p, n)
                s <= g(b) - g(a)
            }
        }
        sup_le_of_upper_bound(lower_sum_set(f, a, b), integral(f, a, b), g(b) - g(a))
        integral(f, a, b) <= g(b) - g(a)
        // Every upper sum is at least g(b) - g(a), so g(b) - g(a) is a lower
        // bound of the upper sums and the infimum lies above it.
        forall(s: Real) {
            if upper_sum_set(f, a, b).contains(s) {
                upper_sum_set(f, a, b).contains(s) = upper_sum_contains(f, a, b, s)
                upper_sum_contains(f, a, b, s)
                let (p: Nat -> Real, n: Nat) satisfy {
                    is_partition(p, a, b, n) and s = upper_sum(f, p, n)
                }
                is_partition(p, a, b, n)
                g_diff_le_upper_sum(f, g, a, b, ub, p, n)
                g(b) - g(a) <= upper_sum(f, p, n)
                s = upper_sum(f, p, n)
                g(b) - g(a) <= s
            }
        }
        is_set_lower_bound(upper_sum_set(f, a, b), g(b) - g(a))
        set_lower_bound_le_infimum(upper_sum_set(f, a, b), integral(f, a, b), g(b) - g(a))
        g(b) - g(a) <= integral(f, a, b)
        lte_antisymm[Real](integral(f, a, b), g(b) - g(a))
        integral(f, a, b) = g(b) - g(a)
    }
}

// ---------------------------------------------------------------------------
// The integral of the exponential
// ---------------------------------------------------------------------------

/// The integral of Real.exp over [a, b] is b.exp - a.exp, once Real.exp is known to
/// be integrable on [a, b].
theorem integral_exp_ftc2(a: Real, b: Real) {
    a <= b and is_integrable(Real.exp, a, b)
    implies
    integral(Real.exp, a, b) = b.exp - a.exp
} by {
    if a <= b and is_integrable(Real.exp, a, b) {
        exp_continuous
        continuous(Real.exp)
        exp_is_derivative_fn
        is_derivative_fn(Real.exp, Real.exp)
        exp_lower_bound_on(a, b)
        exp_upper_bound_on(a, b)
        ftc2_general(Real.exp, Real.exp, a, b, a.exp, b.exp)
        integral(Real.exp, a, b) = b.exp - a.exp
    }
}


// ---------------------------------------------------------------------------
// Small ring helpers
// ---------------------------------------------------------------------------

/// Ring helper: a + (b - a) = b.
theorem add_sub_cancel_shift(a: Real, b: Real) {
    a + (b - a) = b
} by {
    b - a = b + -a
    a + (b - a) = a + (b + -a)
    add_assoc(a, b, -a)
    (a + b) + -a = a + (b + -a)
    a + (b + -a) = (a + b) + -a
    add_comm(a, b)
    a + b = b + a
    (a + b) + -a = (b + a) + -a
    add_assoc(b, a, -a)
    (b + a) + -a = b + (a + -a)
    a + -a = Real.0
    b + (a + -a) = b + Real.0
    b + Real.0 = b
    a + (b - a) = b
}

/// Ring helper: (a + x) - (a + y) = x - y.
theorem sub_add_cancel_shift(a: Real, x: Real, y: Real) {
    (a + x) - (a + y) = x - y
} by {
    (a + x) - (a + y) = (a + x) + -(a + y)
    neg_distrib(a, y)
    -(a + y) = -a + -y
    (a + x) + -(a + y) = (a + x) + (-a + -y)
    add_assoc(a + x, -a, -y)
    (a + x) + (-a + -y) = (a + x) + -a + -y
    add_comm(a, x)
    a + x = x + a
    (a + x) + -a = (x + a) + -a
    add_assoc(x, a, -a)
    (x + a) + -a = x + (a + -a)
    a + -a = Real.0
    x + (a + -a) = x + Real.0
    x + Real.0 = x
    (a + x) + -a = x
    (a + x) + -a + -y = x + -y
    (a + x) + (-a + -y) = x + -y
    x - y = x + -y
    (a + x) - (a + y) = x - y
}

/// Ring helper: (u + 1) * h - u * h = h.
theorem add_one_mul_sub(u: Real, h: Real) {
    (u + Real.1) * h - u * h = h
} by {
    mul_sub_distrib_left(u + Real.1, u, h)
    (u + Real.1 - u) * h = (u + Real.1) * h - u * h
    (u + Real.1) * h - u * h = (u + Real.1 - u) * h
    add_comm(u, Real.1)
    u + Real.1 = Real.1 + u
    u + Real.1 - u = Real.1 + u - u
    add_assoc(Real.1, u, -u)
    (Real.1 + u) + -u = Real.1 + (u + -u)
    Real.1 + u - u = Real.1 + (u + -u)
    u + -u = Real.0
    Real.1 + (u + -u) = Real.1 + Real.0
    Real.1 + Real.0 = Real.1
    u + Real.1 - u = Real.1
    (u + Real.1 - u) * h = Real.1 * h
    Real.1 * h = h
    (u + Real.1) * h - u * h = h
}

/// Multiplying a fraction by a real: (a / b) * c = (a * c) / b.
theorem mul_frac_right(a: Real, b: Real, c: Real) {
    b != Real.0 implies (a / b) * c = (a * c) / b
} by {
    if b != Real.0 {
        Real.1 != Real.0
        mul_div(a, b, c, Real.1)
        (a / b) * (c / Real.1) = (a * c) / (b * Real.1)
        c / Real.1 = c
        b * Real.1 = b
        (a / b) * c = (a * c) / b
    }
}

// ---------------------------------------------------------------------------
// The exponential is integrable on every closed interval
// ---------------------------------------------------------------------------
//
// Because Real.exp is increasing, its supremum and infimum over a subinterval
// [x, y] are attained at the endpoints (interval_sup_exp_right and
// interval_inf_exp_left).  Over the uniform partition of [a, b] into n + 1
// equal subintervals the upper and lower Darboux sums therefore telescope to
// ((b - a) / (n + 1)) * (b.exp - a.exp) apart (uniform_upper_minus_lower).
// The supremum L of the lower sums and the infimum U of the upper sums both
// exist (completeness), satisfy L <= b.exp - a.exp <= U by the
// mean-value-theorem sandwich, and satisfy U - L <= ((b - a) * (b.exp -
// a.exp)) / (n + 1) for every n; the Archimedean property then forces
// U = L, which is exactly integrability.

/// The supremum of Real.exp over [x, y] is attained at the right endpoint.
theorem interval_sup_exp_right(x: Real, y: Real) {
    x <= y implies interval_sup(Real.exp, x, y) = y.exp
} by {
    if x <= y {
        forall(t: Real) {
            if interval_contains(x, y, t) {
                interval_contains_right(x, y, t)
                t <= y
                exp_lte_mono(t, y)
                t.exp <= y.exp
            }
        }
        image_upper_bound(Real.exp, x, y, y.exp)
        is_set_upper_bound(interval_image(Real.exp, x, y), y.exp)
        interval_set_contains_right(x, y)
        interval_set(x, y).contains(y)
        maps_into_set_image(interval_set(x, y), Real.exp, y)
        function_image(Real.exp, interval_set(x, y)).contains(y.exp)
        interval_image(Real.exp, x, y).contains(y.exp)
        exists(b: Real) {
            is_set_upper_bound(interval_image(Real.exp, x, y), b)
        }
        has_upper_bound(interval_image(Real.exp, x, y))
        interval_set_contains_left(x, y)
        is_nonempty(interval_set(x, y))
        is_nonempty(interval_set(x, y)) and has_upper_bound(interval_image(Real.exp, x, y))
        interval_sup_spec(Real.exp, x, y)
        is_set_supremum(interval_image(Real.exp, x, y), interval_sup(Real.exp, x, y))
        forall(b: Real) {
            if is_set_upper_bound(interval_image(Real.exp, x, y), b) {
                set_upper_bound_contains_le(interval_image(Real.exp, x, y), b, y.exp)
                y.exp <= b
            }
        }
        is_set_upper_bound(interval_image(Real.exp, x, y), y.exp) and forall(b: Real) {
            is_set_upper_bound(interval_image(Real.exp, x, y), b) implies y.exp <= b
        }
        is_set_supremum(interval_image(Real.exp, x, y), y.exp)
        set_supremum_unique(interval_image(Real.exp, x, y), interval_sup(Real.exp, x, y), y.exp)
        interval_sup(Real.exp, x, y) = y.exp
    }
}

/// The infimum of Real.exp over [x, y] is attained at the left endpoint.
theorem interval_inf_exp_left(x: Real, y: Real) {
    x <= y implies interval_inf(Real.exp, x, y) = x.exp
} by {
    if x <= y {
        forall(t: Real) {
            if interval_contains(x, y, t) {
                interval_contains_left(x, y, t)
                x <= t
                exp_lte_mono(x, t)
                x.exp <= t.exp
            }
        }
        image_lower_bound(Real.exp, x, y, x.exp)
        is_set_lower_bound(interval_image(Real.exp, x, y), x.exp)
        interval_set_contains_left(x, y)
        interval_set(x, y).contains(x)
        maps_into_set_image(interval_set(x, y), Real.exp, x)
        function_image(Real.exp, interval_set(x, y)).contains(x.exp)
        interval_image(Real.exp, x, y).contains(x.exp)
        exists(b: Real) {
            is_set_lower_bound(interval_image(Real.exp, x, y), b)
        }
        has_lower_bound(interval_image(Real.exp, x, y))
        is_nonempty(interval_set(x, y))
        is_nonempty(interval_set(x, y)) and has_lower_bound(interval_image(Real.exp, x, y))
        interval_inf_spec(Real.exp, x, y)
        is_set_infimum(interval_image(Real.exp, x, y), interval_inf(Real.exp, x, y))
        set_infimum_is_lower_bound(interval_image(Real.exp, x, y), interval_inf(Real.exp, x, y))
        is_set_lower_bound(interval_image(Real.exp, x, y), interval_inf(Real.exp, x, y))
        set_lower_bound_contains_le(interval_image(Real.exp, x, y), interval_inf(Real.exp, x, y), x.exp)
        interval_inf(Real.exp, x, y) <= x.exp
        forall(b: Real) {
            if is_set_lower_bound(interval_image(Real.exp, x, y), b) {
                set_lower_bound_contains_le(interval_image(Real.exp, x, y), b, x.exp)
                b <= x.exp
            }
        }
        is_set_lower_bound(interval_image(Real.exp, x, y), x.exp) and forall(b: Real) {
            is_set_lower_bound(interval_image(Real.exp, x, y), b) implies b <= x.exp
        }
        is_set_infimum(interval_image(Real.exp, x, y), x.exp)
        set_infimum_unique(interval_image(Real.exp, x, y), interval_inf(Real.exp, x, y), x.exp)
        interval_inf(Real.exp, x, y) = x.exp
    }
}

/// A nonnegative real divided by a positive real is nonnegative.
theorem div_nonneg_pos_denom(a: Real, d: Real) {
    Real.0 <= a and d > Real.0 implies Real.0 <= a / d
} by {
    if Real.0 <= a and d > Real.0 {
        gt_zero_imp_pos(d)
        d.is_positive
        inverse_pos(d)
        d.inverse.is_positive
        not d.inverse.is_negative
        lte_mul_nonneg_right(Real.0, a, d.inverse)
        Real.0 * d.inverse <= a * d.inverse
        Real.0 * d.inverse = Real.0
        Real.0 <= a * d.inverse
        a / d = a * d.inverse
        Real.0 <= a / d
    }
}

/// Converting natural numbers to reals is nondecreasing.
theorem from_nat_lte_mono(m: Nat, n: Nat) {
    m <= n implies from_nat[Real](m) <= from_nat[Real](n)
} by {
    if m <= n {
        rat_from_nat_lte_of_nat_lte(m, n)
        Rat.from_nat(m) <= Rat.from_nat(n)
        from_rat_maintains_lte(Rat.from_nat(m), Rat.from_nat(n))
        Real.from_rat(Rat.from_nat(m)) <= Real.from_rat(Rat.from_nat(n))
        from_nat_is_from_rat(m)
        from_nat[Real](m) = Real.from_rat(Rat.from_nat(m))
        from_nat_is_from_rat(n)
        from_nat[Real](n) = Real.from_rat(Rat.from_nat(n))
        from_nat[Real](m) <= from_nat[Real](n)
    }
}

/// The uniform partition of [a, b] into n + 1 equal subintervals.
define uniform_partition(a: Real, b: Real, n: Nat, i: Nat) -> Real {
    a + from_nat[Real](i) * ((b - a) / from_nat[Real](n.suc))
}

/// The uniform partition starts at a.
theorem uniform_partition_start(a: Real, b: Real, n: Nat) {
    uniform_partition(a, b, n, Nat.0) = a
} by {
    uniform_partition(a, b, n, Nat.0) = a + from_nat[Real](Nat.0) * ((b - a) / from_nat[Real](n.suc))
    from_nat[Real](Nat.0) = Real.0
    from_nat[Real](Nat.0) * ((b - a) / from_nat[Real](n.suc)) = Real.0
    a + Real.0 = a
    uniform_partition(a, b, n, Nat.0) = a
}

/// The uniform partition ends at b.
theorem uniform_partition_end(a: Real, b: Real, n: Nat) {
    uniform_partition(a, b, n, n.suc) = b
} by {
    uniform_partition(a, b, n, n.suc) = a + from_nat[Real](n.suc) * ((b - a) / from_nat[Real](n.suc))
    from_nat_suc_pos_real(n)
    from_nat[Real](n.suc) > Real.0
    from_nat[Real](n.suc) != Real.0
    div_mul_cancel_denominator(b - a, from_nat[Real](n.suc))
    ((b - a) / from_nat[Real](n.suc)) * from_nat[Real](n.suc) = b - a
    from_nat[Real](n.suc) * ((b - a) / from_nat[Real](n.suc)) = b - a
    add_sub_cancel_shift(a, b)
    a + (b - a) = b
    uniform_partition(a, b, n, n.suc) = b
}

/// The uniform partition is nondecreasing.
theorem uniform_partition_monotone(a: Real, b: Real, n: Nat, i: Nat, j: Nat) {
    a <= b and i <= j and j <= n.suc
    implies uniform_partition(a, b, n, i) <= uniform_partition(a, b, n, j)
} by {
    if a <= b and i <= j and j <= n.suc {
        let h = (b - a) / from_nat[Real](n.suc)
        from_nat_suc_pos_real(n)
        from_nat[Real](n.suc) > Real.0
        sub_nonneg(a, b)
        Real.0 <= b - a
        div_nonneg_pos_denom(b - a, from_nat[Real](n.suc))
        Real.0 <= h
        from_nat_lte_mono(i, j)
        from_nat[Real](i) <= from_nat[Real](j)
        mul_le_mul_of_nonneg_right(from_nat[Real](i), from_nat[Real](j), h)
        from_nat[Real](i) * h <= from_nat[Real](j) * h
        add_le_add_right[Real](from_nat[Real](i) * h, from_nat[Real](j) * h, a)
        from_nat[Real](i) * h + a <= from_nat[Real](j) * h + a
        a + from_nat[Real](i) * h <= a + from_nat[Real](j) * h
        uniform_partition(a, b, n, i) = a + from_nat[Real](i) * h
        uniform_partition(a, b, n, j) = a + from_nat[Real](j) * h
        uniform_partition(a, b, n, i) <= uniform_partition(a, b, n, j)
    }
}

/// The uniform partition is a partition of [a, b] of length n + 1.
theorem uniform_partition_is_partition(a: Real, b: Real, n: Nat) {
    a <= b implies is_partition(uniform_partition(a, b, n), a, b, n.suc)
} by {
    if a <= b {
        uniform_partition_start(a, b, n)
        uniform_partition(a, b, n, Nat.0) = a
        uniform_partition_end(a, b, n)
        uniform_partition(a, b, n, n.suc) = b
        forall(i: Nat, j: Nat) {
            if i <= j and j <= n.suc {
                uniform_partition_monotone(a, b, n, i, j)
                uniform_partition(a, b, n, i) <= uniform_partition(a, b, n, j)
            }
        }
        partition_monotone(uniform_partition(a, b, n), n.suc)
        is_partition(uniform_partition(a, b, n), a, b, n.suc)
    }
}

/// Each subinterval of the uniform partition has width (b - a) / (n + 1).
theorem uniform_partition_width(a: Real, b: Real, n: Nat, i: Nat) {
    i < n.suc implies
    uniform_partition(a, b, n, i + 1) - uniform_partition(a, b, n, i) =
        (b - a) / from_nat[Real](n.suc)
} by {
    if i < n.suc {
        uniform_partition(a, b, n, i + 1) = a + from_nat[Real](i + 1) * ((b - a) / from_nat[Real](n.suc))
        uniform_partition(a, b, n, i) = a + from_nat[Real](i) * ((b - a) / from_nat[Real](n.suc))
        from_nat_add[Real](i, Nat.1)
        from_nat[Real](i + Nat.1) = from_nat[Real](i) + from_nat[Real](Nat.1)
        from_nat[Real](i + 1) = from_nat[Real](i) + from_nat[Real](Nat.1)
        from_nat[Real](Nat.1) = Real.1
        from_nat[Real](i + 1) = from_nat[Real](i) + Real.1
        uniform_partition(a, b, n, i + 1) - uniform_partition(a, b, n, i) =
            (a + (from_nat[Real](i) + Real.1) * ((b - a) / from_nat[Real](n.suc))) -
            (a + from_nat[Real](i) * ((b - a) / from_nat[Real](n.suc)))
        sub_add_cancel_shift(a, (from_nat[Real](i) + Real.1) * ((b - a) / from_nat[Real](n.suc)),
            from_nat[Real](i) * ((b - a) / from_nat[Real](n.suc)))
        (a + (from_nat[Real](i) + Real.1) * ((b - a) / from_nat[Real](n.suc))) -
            (a + from_nat[Real](i) * ((b - a) / from_nat[Real](n.suc))) =
            (from_nat[Real](i) + Real.1) * ((b - a) / from_nat[Real](n.suc)) -
            from_nat[Real](i) * ((b - a) / from_nat[Real](n.suc))
        add_one_mul_sub(from_nat[Real](i), (b - a) / from_nat[Real](n.suc))
        (from_nat[Real](i) + Real.1) * ((b - a) / from_nat[Real](n.suc)) -
            from_nat[Real](i) * ((b - a) / from_nat[Real](n.suc)) =
            (b - a) / from_nat[Real](n.suc)
        uniform_partition(a, b, n, i + 1) - uniform_partition(a, b, n, i) =
            (b - a) / from_nat[Real](n.suc)
    }
}

/// The difference of the upper and lower Darboux sums of Real.exp over the uniform
/// partition is the mesh times the endpoint difference.
theorem uniform_upper_minus_lower(a: Real, b: Real, n: Nat) {
    a <= b implies
    upper_sum(Real.exp, uniform_partition(a, b, n), n.suc) -
        lower_sum(Real.exp, uniform_partition(a, b, n), n.suc) =
        ((b - a) / from_nat[Real](n.suc)) * (b.exp - a.exp)
} by {
    if a <= b {
        uniform_partition_is_partition(a, b, n)
        is_partition(uniform_partition(a, b, n), a, b, n.suc)
        forall(i: Nat) {
            if i < n.suc {
                uniform_partition_width(a, b, n, i)
                uniform_partition(a, b, n, i + 1) - uniform_partition(a, b, n, i) =
                    (b - a) / from_nat[Real](n.suc)
                i <= i + 1
                lt_imp_lte_suc(i, n.suc)
                i + 1 <= n.suc
                partition_mono(uniform_partition(a, b, n), a, b, n.suc, i, i + 1)
                uniform_partition(a, b, n, i) <= uniform_partition(a, b, n, i + 1)
                interval_sup_exp_right(uniform_partition(a, b, n, i), uniform_partition(a, b, n, i + 1))
                interval_sup(Real.exp, uniform_partition(a, b, n, i), uniform_partition(a, b, n, i + 1)) =
                    (uniform_partition(a, b, n, i + 1)).exp
                interval_inf_exp_left(uniform_partition(a, b, n, i), uniform_partition(a, b, n, i + 1))
                interval_inf(Real.exp, uniform_partition(a, b, n, i), uniform_partition(a, b, n, i + 1)) =
                    (uniform_partition(a, b, n, i)).exp
                partition_step_upper(Real.exp, uniform_partition(a, b, n), i) =
                    interval_sup(Real.exp, uniform_partition(a, b, n, i), uniform_partition(a, b, n, i + 1)) *
                        diff_step(uniform_partition(a, b, n), i)
                diff_step(uniform_partition(a, b, n), i) =
                    uniform_partition(a, b, n, i + 1) - uniform_partition(a, b, n, i)
                partition_step_upper(Real.exp, uniform_partition(a, b, n), i) =
                    (uniform_partition(a, b, n, i + 1)).exp * ((b - a) / from_nat[Real](n.suc))
                partition_step_lower(Real.exp, uniform_partition(a, b, n), i) =
                    interval_inf(Real.exp, uniform_partition(a, b, n, i), uniform_partition(a, b, n, i + 1)) *
                        diff_step(uniform_partition(a, b, n), i)
                partition_step_lower(Real.exp, uniform_partition(a, b, n), i) =
                    (uniform_partition(a, b, n, i)).exp * ((b - a) / from_nat[Real](n.suc))
                partition_step_upper(Real.exp, uniform_partition(a, b, n), i) -
                    partition_step_lower(Real.exp, uniform_partition(a, b, n), i) =
                    ((uniform_partition(a, b, n, i + 1)).exp - (uniform_partition(a, b, n, i)).exp) *
                        ((b - a) / from_nat[Real](n.suc))
                sub_seq(partition_step_upper(Real.exp, uniform_partition(a, b, n)),
                    partition_step_lower(Real.exp, uniform_partition(a, b, n)), i) =
                    partition_step_upper(Real.exp, uniform_partition(a, b, n), i) -
                    partition_step_lower(Real.exp, uniform_partition(a, b, n), i)
                diff_step(partition_g_values(Real.exp, uniform_partition(a, b, n)), i) =
                    partition_g_values(Real.exp, uniform_partition(a, b, n), i + 1) -
                    partition_g_values(Real.exp, uniform_partition(a, b, n), i)
                partition_g_values(Real.exp, uniform_partition(a, b, n), i + 1) =
                    (uniform_partition(a, b, n, i + 1)).exp
                partition_g_values(Real.exp, uniform_partition(a, b, n), i) =
                    (uniform_partition(a, b, n, i)).exp
                diff_step(partition_g_values(Real.exp, uniform_partition(a, b, n)), i) =
                    (uniform_partition(a, b, n, i + 1)).exp - (uniform_partition(a, b, n, i)).exp
                sub_seq(partition_step_upper(Real.exp, uniform_partition(a, b, n)),
                    partition_step_lower(Real.exp, uniform_partition(a, b, n)), i) =
                    ((b - a) / from_nat[Real](n.suc)) *
                        ((uniform_partition(a, b, n, i + 1)).exp - (uniform_partition(a, b, n, i)).exp)
                mul_fn((b - a) / from_nat[Real](n.suc),
                    diff_step(partition_g_values(Real.exp, uniform_partition(a, b, n))), i) =
                    ((b - a) / from_nat[Real](n.suc)) *
                        diff_step(partition_g_values(Real.exp, uniform_partition(a, b, n)), i)
                sub_seq(partition_step_upper(Real.exp, uniform_partition(a, b, n)),
                    partition_step_lower(Real.exp, uniform_partition(a, b, n)), i) =
                    mul_fn((b - a) / from_nat[Real](n.suc),
                        diff_step(partition_g_values(Real.exp, uniform_partition(a, b, n))), i)
            }
        }
        partial_pointwise_eq(sub_seq(partition_step_upper(Real.exp, uniform_partition(a, b, n)),
                partition_step_lower(Real.exp, uniform_partition(a, b, n))),
            mul_fn((b - a) / from_nat[Real](n.suc),
                diff_step(partition_g_values(Real.exp, uniform_partition(a, b, n)))), n.suc)
        partial(sub_seq(partition_step_upper(Real.exp, uniform_partition(a, b, n)),
                partition_step_lower(Real.exp, uniform_partition(a, b, n))), n.suc) =
            partial(mul_fn((b - a) / from_nat[Real](n.suc),
                diff_step(partition_g_values(Real.exp, uniform_partition(a, b, n)))), n.suc)
        partial_sub_seq(partition_step_upper(Real.exp, uniform_partition(a, b, n)),
            partition_step_lower(Real.exp, uniform_partition(a, b, n)), n.suc)
        partial(sub_seq(partition_step_upper(Real.exp, uniform_partition(a, b, n)),
                partition_step_lower(Real.exp, uniform_partition(a, b, n))), n.suc) =
            partial(partition_step_upper(Real.exp, uniform_partition(a, b, n)), n.suc) -
            partial(partition_step_lower(Real.exp, uniform_partition(a, b, n)), n.suc)
        partial(partition_step_upper(Real.exp, uniform_partition(a, b, n)), n.suc) -
            partial(partition_step_lower(Real.exp, uniform_partition(a, b, n)), n.suc) =
            partial(mul_fn((b - a) / from_nat[Real](n.suc),
                diff_step(partition_g_values(Real.exp, uniform_partition(a, b, n)))), n.suc)
        partial_scalar_mul((b - a) / from_nat[Real](n.suc),
            diff_step(partition_g_values(Real.exp, uniform_partition(a, b, n))), n.suc)
        partial(mul_fn((b - a) / from_nat[Real](n.suc),
                diff_step(partition_g_values(Real.exp, uniform_partition(a, b, n)))), n.suc) =
            ((b - a) / from_nat[Real](n.suc)) *
                partial(diff_step(partition_g_values(Real.exp, uniform_partition(a, b, n))), n.suc)
        partial(partition_step_upper(Real.exp, uniform_partition(a, b, n)), n.suc) -
            partial(partition_step_lower(Real.exp, uniform_partition(a, b, n)), n.suc) =
            ((b - a) / from_nat[Real](n.suc)) *
                partial(diff_step(partition_g_values(Real.exp, uniform_partition(a, b, n))), n.suc)
        telescope(partition_g_values(Real.exp, uniform_partition(a, b, n)), n.suc)
        partial(diff_step(partition_g_values(Real.exp, uniform_partition(a, b, n))), n.suc) =
            partition_g_values(Real.exp, uniform_partition(a, b, n), n.suc) -
            partition_g_values(Real.exp, uniform_partition(a, b, n), Nat.0)
        partition_end(uniform_partition(a, b, n), a, b, n.suc)
        uniform_partition(a, b, n, n.suc) = b
        partition_start(uniform_partition(a, b, n), a, b, n.suc)
        uniform_partition(a, b, n, Nat.0) = a
        partition_g_values(Real.exp, uniform_partition(a, b, n), n.suc) =
            (uniform_partition(a, b, n, n.suc)).exp
        partition_g_values(Real.exp, uniform_partition(a, b, n), n.suc) = b.exp
        partition_g_values(Real.exp, uniform_partition(a, b, n), Nat.0) =
            (uniform_partition(a, b, n, Nat.0)).exp
        partition_g_values(Real.exp, uniform_partition(a, b, n), Nat.0) = a.exp
        partition_g_values(Real.exp, uniform_partition(a, b, n), n.suc) -
            partition_g_values(Real.exp, uniform_partition(a, b, n), Nat.0) = b.exp - a.exp
        partial(diff_step(partition_g_values(Real.exp, uniform_partition(a, b, n))), n.suc) =
            b.exp - a.exp
        partial(partition_step_upper(Real.exp, uniform_partition(a, b, n)), n.suc) -
            partial(partition_step_lower(Real.exp, uniform_partition(a, b, n)), n.suc) =
            ((b - a) / from_nat[Real](n.suc)) * (b.exp - a.exp)
        upper_sum(Real.exp, uniform_partition(a, b, n), n.suc) =
            partial(partition_step_upper(Real.exp, uniform_partition(a, b, n)), n.suc)
        lower_sum(Real.exp, uniform_partition(a, b, n), n.suc) =
            partial(partition_step_lower(Real.exp, uniform_partition(a, b, n)), n.suc)
        upper_sum(Real.exp, uniform_partition(a, b, n), n.suc) -
            lower_sum(Real.exp, uniform_partition(a, b, n), n.suc) =
            ((b - a) / from_nat[Real](n.suc)) * (b.exp - a.exp)
    }
}

/// The lower sums of Real.exp over [a, b] are bounded above by b.exp - a.exp.
theorem exp_lower_sum_set_bounded_above(a: Real, b: Real) {
    a <= b implies is_set_upper_bound(lower_sum_set(Real.exp, a, b), b.exp - a.exp)
} by {
    if a <= b {
        forall(s: Real) {
            if lower_sum_set(Real.exp, a, b).contains(s) {
                lower_sum_set(Real.exp, a, b).contains(s) = lower_sum_contains(Real.exp, a, b, s)
                lower_sum_contains(Real.exp, a, b, s)
                let (p: Nat -> Real, n: Nat) satisfy {
                    is_partition(p, a, b, n) and s = lower_sum(Real.exp, p, n)
                }
                is_partition(p, a, b, n)
                exp_continuous
                continuous(Real.exp)
                exp_is_derivative_fn
                is_derivative_fn(Real.exp, Real.exp)
                exp_lower_bound_on(a, b)
                lower_sum_le_g_diff(Real.exp, Real.exp, a, b, a.exp, p, n)
                lower_sum(Real.exp, p, n) <= b.exp - a.exp
                s = lower_sum(Real.exp, p, n)
                s <= b.exp - a.exp
            }
        }
        is_set_upper_bound(lower_sum_set(Real.exp, a, b), b.exp - a.exp)
    }
}

/// The upper sums of Real.exp over [a, b] are bounded below by b.exp - a.exp.
theorem exp_upper_sum_set_bounded_below(a: Real, b: Real) {
    a <= b implies is_set_lower_bound(upper_sum_set(Real.exp, a, b), b.exp - a.exp)
} by {
    if a <= b {
        forall(s: Real) {
            if upper_sum_set(Real.exp, a, b).contains(s) {
                upper_sum_set(Real.exp, a, b).contains(s) = upper_sum_contains(Real.exp, a, b, s)
                upper_sum_contains(Real.exp, a, b, s)
                let (p: Nat -> Real, n: Nat) satisfy {
                    is_partition(p, a, b, n) and s = upper_sum(Real.exp, p, n)
                }
                is_partition(p, a, b, n)
                exp_continuous
                continuous(Real.exp)
                exp_is_derivative_fn
                is_derivative_fn(Real.exp, Real.exp)
                exp_upper_bound_on(a, b)
                g_diff_le_upper_sum(Real.exp, Real.exp, a, b, b.exp, p, n)
                b.exp - a.exp <= upper_sum(Real.exp, p, n)
                s = upper_sum(Real.exp, p, n)
                b.exp - a.exp <= s
            }
        }
        is_set_lower_bound(upper_sum_set(Real.exp, a, b), b.exp - a.exp)
    }
}

/// The set of lower sums of Real.exp over [a, b] has a supremum.
theorem exp_lower_sum_set_sup_exists(a: Real, b: Real) {
    a <= b implies exists(l: Real) {
        is_set_supremum(lower_sum_set(Real.exp, a, b), l)
    }
} by {
    if a <= b {
        trivial_partition_is_partition(a, b)
        is_partition(trivial_partition(a, b), a, b, Nat.1)
        is_partition(trivial_partition(a, b), a, b, Nat.1) and
            lower_sum(Real.exp, trivial_partition(a, b), Nat.1) = lower_sum(Real.exp, trivial_partition(a, b), Nat.1)
        exists(p: Nat -> Real, n: Nat) {
            is_partition(p, a, b, n) and lower_sum(Real.exp, trivial_partition(a, b), Nat.1) = lower_sum(Real.exp, p, n)
        }
        lower_sum_contains(Real.exp, a, b, lower_sum(Real.exp, trivial_partition(a, b), Nat.1))
        lower_sum_set(Real.exp, a, b).contains(lower_sum(Real.exp, trivial_partition(a, b), Nat.1)) =
            lower_sum_contains(Real.exp, a, b, lower_sum(Real.exp, trivial_partition(a, b), Nat.1))
        lower_sum_set(Real.exp, a, b).contains(lower_sum(Real.exp, trivial_partition(a, b), Nat.1))
        exists(x: Real) {
            lower_sum_set(Real.exp, a, b).contains(x)
        }
        is_nonempty(lower_sum_set(Real.exp, a, b))
        exp_lower_sum_set_bounded_above(a, b)
        is_set_upper_bound(lower_sum_set(Real.exp, a, b), b.exp - a.exp)
        exists(b0: Real) {
            is_set_upper_bound(lower_sum_set(Real.exp, a, b), b0)
        }
        has_upper_bound(lower_sum_set(Real.exp, a, b))
        is_nonempty(lower_sum_set(Real.exp, a, b)) and has_upper_bound(lower_sum_set(Real.exp, a, b))
        completeness(lower_sum_set(Real.exp, a, b))
        let l: Real satisfy {
            is_set_supremum(lower_sum_set(Real.exp, a, b), l)
        }
        exists(l2: Real) {
            is_set_supremum(lower_sum_set(Real.exp, a, b), l2)
        }
    }
}

/// The set of upper sums of Real.exp over [a, b] has an infimum.
theorem exp_upper_sum_set_inf_exists(a: Real, b: Real) {
    a <= b implies exists(u: Real) {
        is_set_infimum(upper_sum_set(Real.exp, a, b), u)
    }
} by {
    if a <= b {
        trivial_partition_is_partition(a, b)
        is_partition(trivial_partition(a, b), a, b, Nat.1)
        is_partition(trivial_partition(a, b), a, b, Nat.1) and
            upper_sum(Real.exp, trivial_partition(a, b), Nat.1) = upper_sum(Real.exp, trivial_partition(a, b), Nat.1)
        exists(p: Nat -> Real, n: Nat) {
            is_partition(p, a, b, n) and upper_sum(Real.exp, trivial_partition(a, b), Nat.1) = upper_sum(Real.exp, p, n)
        }
        upper_sum_contains(Real.exp, a, b, upper_sum(Real.exp, trivial_partition(a, b), Nat.1))
        upper_sum_set(Real.exp, a, b).contains(upper_sum(Real.exp, trivial_partition(a, b), Nat.1)) =
            upper_sum_contains(Real.exp, a, b, upper_sum(Real.exp, trivial_partition(a, b), Nat.1))
        upper_sum_set(Real.exp, a, b).contains(upper_sum(Real.exp, trivial_partition(a, b), Nat.1))
        exists(x: Real) {
            upper_sum_set(Real.exp, a, b).contains(x)
        }
        is_nonempty(upper_sum_set(Real.exp, a, b))
        negate_set_nonempty(upper_sum_set(Real.exp, a, b))
        is_nonempty(negate_set(upper_sum_set(Real.exp, a, b)))
        exp_upper_sum_set_bounded_below(a, b)
        is_set_lower_bound(upper_sum_set(Real.exp, a, b), b.exp - a.exp)
        neg_upper_bound_of_lower(upper_sum_set(Real.exp, a, b), b.exp - a.exp)
        is_set_upper_bound(negate_set(upper_sum_set(Real.exp, a, b)), -(b.exp - a.exp))
        exists(b0: Real) {
            is_set_upper_bound(negate_set(upper_sum_set(Real.exp, a, b)), b0)
        }
        has_upper_bound(negate_set(upper_sum_set(Real.exp, a, b)))
        is_nonempty(negate_set(upper_sum_set(Real.exp, a, b))) and has_upper_bound(negate_set(upper_sum_set(Real.exp, a, b)))
        completeness(negate_set(upper_sum_set(Real.exp, a, b)))
        let sup: Real satisfy {
            is_set_supremum(negate_set(upper_sum_set(Real.exp, a, b)), sup)
        }
        inf_of_neg_sup(upper_sum_set(Real.exp, a, b), sup)
        is_set_infimum(upper_sum_set(Real.exp, a, b), -sup)
        exists(u2: Real) {
            is_set_infimum(upper_sum_set(Real.exp, a, b), u2)
        }
    }
}

/// If d is at most c / (n + 1) for every n, then d is nonpositive.
theorem frac_all_n_imp_lte_zero(c: Real, d: Real) {
    (forall(n: Nat) { d <= c / from_nat[Real](n.suc) })
    implies d <= Real.0
} by {
    if forall(n: Nat) { d <= c / from_nat[Real](n.suc) } {
        if not d <= Real.0 {
            not_lte_imp_gt[Real](d, Real.0)
            Real.0 < d
            gt_zero_imp_pos(d)
            d.is_positive
            if c <= Real.0 {
                d <= c / from_nat[Real](Nat.0.suc)
                Nat.0.suc = Nat.1
                from_nat[Real](Nat.0.suc) = from_nat[Real](Nat.1)
                from_nat[Real](Nat.1) = Real.1
                from_nat[Real](Nat.0.suc) = Real.1
                c / from_nat[Real](Nat.0.suc) = c / Real.1
                c / Real.1 = c
                d <= c
                lte_trans[Real](d, c, Real.0)
                d <= Real.0
                lte_lt_trans(d, Real.0, d)
                d < d
                not_lt_self(d)
                false
            } else {
                not c <= Real.0
                not_lte_imp_gt[Real](c, Real.0)
                Real.0 < c
                gt_zero_imp_pos(c)
                c.is_positive
                pos_gt_zero(d)
                d > Real.0
                inverse_pos(d)
                d.inverse.is_positive
                mul_pos_pos(c, d.inverse)
                (c * d.inverse).is_positive
                pos_gt_zero(c * d.inverse)
                c * d.inverse > Real.0
                c / d = c * d.inverse
                c / d > Real.0
                gt_zero_imp_pos(c / d)
                (c / d).is_positive
                exists_nat_gt(c / d)
                let n: Nat satisfy {
                    Real.from_rat(Rat.from_nat(n)) > c / d
                }
                from_nat_is_from_rat(n)
                from_nat[Real](n) = Real.from_rat(Rat.from_nat(n))
                from_nat[Real](n) > c / d
                c / d < from_nat[Real](n)
                n <= n.suc
                from_nat_lte_mono(n, n.suc)
                from_nat[Real](n) <= from_nat[Real](n.suc)
                lt_of_lt_of_lte(c / d, from_nat[Real](n), from_nat[Real](n.suc))
                c / d < from_nat[Real](n.suc)
                lt_mul_pos_left(c / d, from_nat[Real](n.suc), d)
                d * (c / d) < d * from_nat[Real](n.suc)
                d != Real.0
                div_mul_cancel_denominator(c, d)
                ((c / d) * d) = c
                d * (c / d) = c
                c < d * from_nat[Real](n.suc)
                from_nat_suc_pos_real(n)
                from_nat[Real](n.suc) > Real.0
                inverse_pos(from_nat[Real](n.suc))
                from_nat[Real](n.suc).inverse.is_positive
                lt_mul_pos_left(c, d * from_nat[Real](n.suc), from_nat[Real](n.suc).inverse)
                from_nat[Real](n.suc).inverse * c < from_nat[Real](n.suc).inverse * (d * from_nat[Real](n.suc))
                from_nat[Real](n.suc).inverse * c = c * from_nat[Real](n.suc).inverse
                add_comm(d, from_nat[Real](n.suc))
                d * from_nat[Real](n.suc) = from_nat[Real](n.suc) * d
                from_nat[Real](n.suc).inverse * (d * from_nat[Real](n.suc)) =
                    from_nat[Real](n.suc).inverse * (from_nat[Real](n.suc) * d)
                from_nat[Real](n.suc).inverse * (from_nat[Real](n.suc) * d) =
                    (from_nat[Real](n.suc).inverse * from_nat[Real](n.suc)) * d
                mul_inverse(from_nat[Real](n.suc))
                from_nat[Real](n.suc) * from_nat[Real](n.suc).inverse = Real.1
                from_nat[Real](n.suc).inverse * from_nat[Real](n.suc) = Real.1
                (from_nat[Real](n.suc).inverse * from_nat[Real](n.suc)) * d = Real.1 * d
                Real.1 * d = d
                from_nat[Real](n.suc).inverse * (d * from_nat[Real](n.suc)) = d
                c * from_nat[Real](n.suc).inverse < d
                c / from_nat[Real](n.suc) = c * from_nat[Real](n.suc).inverse
                c / from_nat[Real](n.suc) < d
                d <= c / from_nat[Real](n.suc)
                lt_of_lte_of_lt(d, c / from_nat[Real](n.suc), d)
                d < d
                not_lt_self(d)
                false
            }
        }
        d <= Real.0
    }
}

/// If a nonnegative d is at most c / (n + 1) for every n, then d is zero.
theorem nonneg_frac_all_n_imp_zero(c: Real, d: Real) {
    Real.0 <= d and (forall(n: Nat) { d <= c / from_nat[Real](n.suc) })
    implies d = Real.0
} by {
    if Real.0 <= d and (forall(n: Nat) { d <= c / from_nat[Real](n.suc) }) {
        frac_all_n_imp_lte_zero(c, d)
        d <= Real.0
        lte_antisymm[Real](d, Real.0)
        d = Real.0
    }
}

/// The exponential is integrable on every closed interval [a, b].
theorem exp_integrable(a: Real, b: Real) {
    a <= b implies is_integrable(Real.exp, a, b)
} by {
    if a <= b {
        exp_lower_sum_set_sup_exists(a, b)
        let l: Real satisfy {
            is_set_supremum(lower_sum_set(Real.exp, a, b), l)
        }
        exp_upper_sum_set_inf_exists(a, b)
        let u: Real satisfy {
            is_set_infimum(upper_sum_set(Real.exp, a, b), u)
        }
        // The sandwich l <= b.exp - a.exp <= u from the mean value theorem.
        exp_lower_sum_set_bounded_above(a, b)
        is_set_upper_bound(lower_sum_set(Real.exp, a, b), b.exp - a.exp)
        set_supremum_le_upper_bound(lower_sum_set(Real.exp, a, b), l, b.exp - a.exp)
        l <= b.exp - a.exp
        exp_upper_sum_set_bounded_below(a, b)
        is_set_lower_bound(upper_sum_set(Real.exp, a, b), b.exp - a.exp)
        set_lower_bound_le_infimum(upper_sum_set(Real.exp, a, b), u, b.exp - a.exp)
        b.exp - a.exp <= u
        lte_trans[Real](l, b.exp - a.exp, u)
        l <= u
        sub_nonneg(l, u)
        Real.0 <= u - l
        // For every n the uniform partition forces u - l below c / (n + 1).
        forall(n: Nat) {
            uniform_upper_minus_lower(a, b, n)
            upper_sum(Real.exp, uniform_partition(a, b, n), n.suc) -
                lower_sum(Real.exp, uniform_partition(a, b, n), n.suc) =
                ((b - a) / from_nat[Real](n.suc)) * (b.exp - a.exp)
            uniform_partition_is_partition(a, b, n)
            is_partition(uniform_partition(a, b, n), a, b, n.suc)
            is_partition(uniform_partition(a, b, n), a, b, n.suc) and
                lower_sum(Real.exp, uniform_partition(a, b, n), n.suc) =
                    lower_sum(Real.exp, uniform_partition(a, b, n), n.suc)
            exists(p0: Nat -> Real, n0: Nat) {
                is_partition(p0, a, b, n0) and
                    lower_sum(Real.exp, uniform_partition(a, b, n), n.suc) = lower_sum(Real.exp, p0, n0)
            }
            lower_sum_contains(Real.exp, a, b, lower_sum(Real.exp, uniform_partition(a, b, n), n.suc))
            lower_sum_set(Real.exp, a, b).contains(lower_sum(Real.exp, uniform_partition(a, b, n), n.suc)) =
                lower_sum_contains(Real.exp, a, b, lower_sum(Real.exp, uniform_partition(a, b, n), n.suc))
            lower_sum_set(Real.exp, a, b).contains(lower_sum(Real.exp, uniform_partition(a, b, n), n.suc))
            set_member_le_supremum(lower_sum_set(Real.exp, a, b), l,
                lower_sum(Real.exp, uniform_partition(a, b, n), n.suc))
            lower_sum(Real.exp, uniform_partition(a, b, n), n.suc) <= l
            is_partition(uniform_partition(a, b, n), a, b, n.suc) and
                upper_sum(Real.exp, uniform_partition(a, b, n), n.suc) =
                    upper_sum(Real.exp, uniform_partition(a, b, n), n.suc)
            exists(p1: Nat -> Real, n1: Nat) {
                is_partition(p1, a, b, n1) and
                    upper_sum(Real.exp, uniform_partition(a, b, n), n.suc) = upper_sum(Real.exp, p1, n1)
            }
            upper_sum_contains(Real.exp, a, b, upper_sum(Real.exp, uniform_partition(a, b, n), n.suc))
            upper_sum_set(Real.exp, a, b).contains(upper_sum(Real.exp, uniform_partition(a, b, n), n.suc)) =
                upper_sum_contains(Real.exp, a, b, upper_sum(Real.exp, uniform_partition(a, b, n), n.suc))
            upper_sum_set(Real.exp, a, b).contains(upper_sum(Real.exp, uniform_partition(a, b, n), n.suc))
            set_infimum_is_lower_bound(upper_sum_set(Real.exp, a, b), u)
            is_set_lower_bound(upper_sum_set(Real.exp, a, b), u)
            set_lower_bound_contains_le(upper_sum_set(Real.exp, a, b), u,
                upper_sum(Real.exp, uniform_partition(a, b, n), n.suc))
            u <= upper_sum(Real.exp, uniform_partition(a, b, n), n.suc)
            neg_lte_flip(lower_sum(Real.exp, uniform_partition(a, b, n), n.suc), l)
            -l <= -lower_sum(Real.exp, uniform_partition(a, b, n), n.suc)
            add_le_add(u, upper_sum(Real.exp, uniform_partition(a, b, n), n.suc),
                -l, -lower_sum(Real.exp, uniform_partition(a, b, n), n.suc))
            u + -l <= upper_sum(Real.exp, uniform_partition(a, b, n), n.suc) +
                -lower_sum(Real.exp, uniform_partition(a, b, n), n.suc)
            u - l = u + -l
            upper_sum(Real.exp, uniform_partition(a, b, n), n.suc) -
                lower_sum(Real.exp, uniform_partition(a, b, n), n.suc) =
                upper_sum(Real.exp, uniform_partition(a, b, n), n.suc) +
                -lower_sum(Real.exp, uniform_partition(a, b, n), n.suc)
            u - l <= upper_sum(Real.exp, uniform_partition(a, b, n), n.suc) -
                lower_sum(Real.exp, uniform_partition(a, b, n), n.suc)
            u - l <= ((b - a) / from_nat[Real](n.suc)) * (b.exp - a.exp)
            from_nat_suc_pos_real(n)
            from_nat[Real](n.suc) > Real.0
            from_nat[Real](n.suc) != Real.0
            mul_frac_right(b - a, from_nat[Real](n.suc), b.exp - a.exp)
            ((b - a) / from_nat[Real](n.suc)) * (b.exp - a.exp) =
                ((b - a) * (b.exp - a.exp)) / from_nat[Real](n.suc)
            u - l <= ((b - a) * (b.exp - a.exp)) / from_nat[Real](n.suc)
        }
        nonneg_frac_all_n_imp_zero((b - a) * (b.exp - a.exp), u - l)
        u - l = Real.0
        sub_zero_imp_eq(u, l)
        u = l
        is_set_infimum(upper_sum_set(Real.exp, a, b), l)
        is_set_supremum(lower_sum_set(Real.exp, a, b), l) and is_set_infimum(upper_sum_set(Real.exp, a, b), l)
        exists(m: Real) {
            is_set_supremum(lower_sum_set(Real.exp, a, b), m) and is_set_infimum(upper_sum_set(Real.exp, a, b), m)
        }
        is_integrable(Real.exp, a, b)
    }
}

/// The integral of Real.exp over [a, b] is b.exp - a.exp.
theorem integral_exp(a: Real, b: Real) {
    a <= b implies integral(Real.exp, a, b) = b.exp - a.exp
} by {
    if a <= b {
        exp_integrable(a, b)
        is_integrable(Real.exp, a, b)
        integral_exp_ftc2(a, b)
        integral(Real.exp, a, b) = b.exp - a.exp
    }
}

/// The integral of Real.exp over [0, 1] is e - 1.
theorem integral_exp_unit_value {
    integral(Real.exp, Real.0, Real.1) = Real.e - Real.1
} by {
    Real.1 > Real.0
    lt_imp_lte(Real.0, Real.1)
    Real.0 <= Real.1
    integral_exp(Real.0, Real.1)
    integral(Real.exp, Real.0, Real.1) = (Real.1).exp - (Real.0).exp
    exp_zero
    (Real.0).exp = Real.1
    (Real.1).exp - (Real.0).exp = (Real.1).exp - Real.1
    Real.e = (Real.1).exp
    (Real.1).exp - Real.1 = Real.e - Real.1
    integral(Real.exp, Real.0, Real.1) = Real.e - Real.1
}
