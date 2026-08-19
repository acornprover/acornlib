from order import lte_trans, not_gte_imp_lt, not_lt_imp_gte, not_lte_imp_gt
from real.exp import exp_gt_one_plus_x, exp_increasing, exp_pos, exp_zero
from real.harmonic import real_one_div_pos, real_recip_antitone_pos
from real.log import Real, exp_le_geometric_recip, exp_neg
from real.real_base import lte_add_left, lte_add_right, lt_add_right
from real.real_field import inverse_div

numerals Real

/// Taking the reciprocal twice recovers a nonzero real number.
theorem one_div_one_div(c: Real) {
    c != Real.0 implies Real.1 / (Real.1 / c) = c
} by {
    Real.1 != Real.0
    inverse_div(Real.1, c)
    (Real.1 / c).inverse = c / Real.1
    c / Real.1 = c
    Real.1 / (Real.1 / c) = (Real.1 / c).inverse
    Real.1 / (Real.1 / c) = c
}

/// If `t >= 1`, then `1 - t <= 0`.
theorem one_sub_nonpositive_of_ge_one(t: Real) {
    t >= Real.1 implies Real.1 - t <= Real.0
} by {
    Real.1 <= t
    Real.1 + -t <= t + -t
    t + -t = Real.0
    Real.1 - t <= Real.0
}

/// For nonnegative `t`, `(-t).exp` is bounded below by `1 - t`.
theorem exp_neg_ge_one_sub(t: Real) {
    t >= Real.0 implies (-t).exp >= Real.1 - t
} by {
    if t >= Real.1 {
        one_sub_nonpositive_of_ge_one(t)
        Real.1 - t <= Real.0
        exp_pos(-t)
        (-t).exp > Real.0
        Real.0 <= (-t).exp
        lte_trans(Real.1 - t, Real.0, (-t).exp)
        Real.1 - t <= (-t).exp
        (-t).exp >= Real.1 - t
    } else {
        not_gte_imp_lt(t, Real.1)
        t < Real.1
        exp_le_geometric_recip(t)
        t.exp <= Real.1 / (Real.1 - t)
        exp_pos(t)
        t.exp > Real.0
        Real.1 - t > Real.0
        real_one_div_pos(Real.1 - t)
        Real.1 / (Real.1 - t) > Real.0
        real_recip_antitone_pos(t.exp, Real.1 / (Real.1 - t))
        Real.1 / (Real.1 / (Real.1 - t)) <= Real.1 / t.exp
        Real.1 - t != Real.0
        one_div_one_div(Real.1 - t)
        Real.1 / (Real.1 / (Real.1 - t)) = Real.1 - t
        exp_neg(t)
        (-t).exp = Real.1 / t.exp
        Real.1 - t <= (-t).exp
        (-t).exp >= Real.1 - t
    }
}

/// The exponential function lies above its tangent line at zero.
theorem exp_ge_one_plus_self(x: Real) {
    x.exp >= Real.1 + x
} by {
    if x > Real.0 {
        exp_gt_one_plus_x(x)
        x.exp > Real.1 + x
        x.exp >= Real.1 + x
    } else {
        if x < Real.0 {
            let t = -x
            t > Real.0
            t >= Real.0
            exp_neg_ge_one_sub(t)
            (-t).exp >= Real.1 - t
            -t = x
            Real.1 - t = Real.1 + x
            x.exp >= Real.1 + x
        } else {
            if not x <= Real.0 {
                not_lte_imp_gt(x, Real.0)
                x > Real.0
                false
            }
            x <= Real.0
            Real.0 <= x
            x = Real.0
            exp_zero
            x.exp = Real.1
            Real.1 + x = Real.1
            x.exp >= Real.1 + x
        }
    }
}

/// The exponential of a nonnegative real is at least one.
theorem exp_ge_one_of_nonneg(x: Real) {
    x >= Real.0 implies x.exp >= Real.1
} by {
    exp_ge_one_plus_self(x)
    x.exp >= Real.1 + x
    lte_add_left(Real.0, x, Real.1)
    Real.1 + Real.0 <= Real.1 + x
    Real.1 + Real.0 = Real.1
    Real.1 <= Real.1 + x
    Real.1 + x <= x.exp
    lte_trans(Real.1, Real.1 + x, x.exp)
    Real.1 <= x.exp
    x.exp >= Real.1
}

/// The exponential of a negative real is less than one.
theorem exp_lt_one_of_neg(x: Real) {
    x < Real.0 implies x.exp < Real.1
} by {
    exp_increasing(x, Real.0)
    x.exp < (Real.0).exp
    exp_zero
    (Real.0).exp = Real.1
    x.exp < Real.1
}

/// The exponential function is monotone.
theorem exp_monotone(x: Real, y: Real) {
    x <= y implies x.exp <= y.exp
} by {
    if x < y {
        exp_increasing(x, y)
        x.exp < y.exp
        x.exp <= y.exp
    } else {
        not_lt_imp_gte(x, y)
        x >= y
        y <= x
        x = y
        x.exp = y.exp
        x.exp <= y.exp
    }
}

/// The exponential of a nonpositive real is at most one.
theorem exp_le_one_of_nonpos(x: Real) {
    x <= Real.0 implies x.exp <= Real.1
} by {
    if x < Real.0 {
        exp_lt_one_of_neg(x)
        x.exp < Real.1
        x.exp <= Real.1
    } else {
        not_lt_imp_gte(x, Real.0)
        x >= Real.0
        Real.0 <= x
        x = Real.0
        exp_zero
        x.exp = Real.1
        x.exp <= Real.1
    }
}

/// For nonnegative `t`, `(-t).exp` is at most one.
theorem exp_neg_le_one_of_nonneg(t: Real) {
    t >= Real.0 implies (-t).exp <= Real.1
} by {
    -t <= Real.0
    exp_le_one_of_nonpos(-t)
    (-t).exp <= Real.1
}

/// For positive `t`, `(-t).exp` is less than one.
theorem exp_neg_lt_one_of_pos(t: Real) {
    t > Real.0 implies (-t).exp < Real.1
} by {
    -t < Real.0
    exp_lt_one_of_neg(-t)
    (-t).exp < Real.1
}

/// The lower tangent-line bound also applies to `(-x).exp`.
theorem exp_neg_ge_one_sub_self(x: Real) {
    (-x).exp >= Real.1 - x
} by {
    exp_ge_one_plus_self(-x)
    (-x).exp >= Real.1 + -x
    Real.1 + -x = Real.1 - x
    (-x).exp >= Real.1 - x
}

/// For nonnegative `t`, `1 - (-t).exp` is bounded above by `t`.
theorem one_sub_exp_neg_le_self(t: Real) {
    t >= Real.0 implies Real.1 - (-t).exp <= t
} by {
    exp_neg_ge_one_sub(t)
    (-t).exp >= Real.1 - t
    Real.1 - t <= (-t).exp
    Real.1 - t + t <= (-t).exp + t
    Real.1 <= (-t).exp + t
    lte_add_right(Real.1, (-t).exp + t, -(-t).exp)
    Real.1 + -(-t).exp <= (-t).exp + t + -(-t).exp
    (-t).exp + t + -(-t).exp = t + (-t).exp + -(-t).exp
    t + (-t).exp + -(-t).exp = t + ((-t).exp + -(-t).exp)
    (-t).exp + -(-t).exp = Real.0
    t + ((-t).exp + -(-t).exp) = t + Real.0
    t + Real.0 = t
    (-t).exp + t + -(-t).exp = t
    Real.1 - (-t).exp <= t
}

/// Every real is at most `x.exp - 1`.
theorem self_le_exp_sub_one(x: Real) {
    x <= x.exp - Real.1
} by {
    exp_ge_one_plus_self(x)
    Real.1 + x <= x.exp
    lte_add_right(Real.1 + x, x.exp, -Real.1)
    Real.1 + x + -Real.1 <= x.exp + -Real.1
    Real.1 + x + -Real.1 = x + Real.1 + -Real.1
    x + Real.1 + -Real.1 = x + (Real.1 + -Real.1)
    Real.1 + -Real.1 = Real.0
    x + (Real.1 + -Real.1) = x + Real.0
    x + Real.0 = x
    Real.1 + x + -Real.1 = x
    x.exp + -Real.1 = x.exp - Real.1
    x <= x.exp - Real.1
}

/// For nonnegative `x`, `x.exp - 1` is nonnegative.
theorem exp_sub_one_nonneg_of_nonneg(x: Real) {
    x >= Real.0 implies x.exp - Real.1 >= Real.0
} by {
    exp_ge_one_of_nonneg(x)
    x.exp >= Real.1
    lte_add_right(Real.1, x.exp, -Real.1)
    Real.1 + -Real.1 <= x.exp + -Real.1
    Real.1 + -Real.1 = Real.0
    x.exp - Real.1 >= Real.0
}

/// For positive `x`, `x.exp - 1` is positive.
theorem exp_sub_one_pos_of_pos(x: Real) {
    x > Real.0 implies x.exp - Real.1 > Real.0
} by {
    exp_gt_one_plus_x(x)
    x.exp > Real.1 + x
    Real.1 + x > Real.1
    x.exp > Real.1
    lt_add_right(Real.1, x.exp, -Real.1)
    Real.1 + -Real.1 < x.exp + -Real.1
    Real.1 + -Real.1 = Real.0
    x.exp - Real.1 > Real.0
}
