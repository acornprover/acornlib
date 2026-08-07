from data.basic.functions import compose
from real.real_field import Real
from real.real_base import bounds_imp_close, lt_add_pos
from order import lt_of_lte_of_lt, not_lte_imp_gt
from real.real_series import is_increasing, image, is_upper_bound
from real.real_seq import converges, limit, eventual_ub
from data.basic.set import Set, image_contains, set_image, set_image_compose, set_image_contains_witness, maps_into_set_image, universal_set_contains_eq
from nat import Nat
from order import is_monotone

// This file defines supremum-related concepts for sets of real numbers.

/// True if the set s is nonempty.
define is_nonempty(s: Set[Real]) -> Bool {
    exists(x: Real) {
        s.contains(x)
    }
}

/// True if b is an upper bound for the set s.
define is_set_upper_bound(s: Set[Real], b: Real) -> Bool {
    forall(x: Real) {
        s.contains(x) implies x <= b
    }
}

/// True if the set s has an upper bound.
define has_upper_bound(s: Set[Real]) -> Bool {
    exists(b: Real) {
        is_set_upper_bound(s, b)
    }
}

/// True if b is a lower bound for the set s.
define is_set_lower_bound(s: Set[Real], b: Real) -> Bool {
    forall(x: Real) {
        s.contains(x) implies b <= x
    }
}

/// True if sup is the supremum (least upper bound) of the set s.
define is_set_supremum(s: Set[Real], sup: Real) -> Bool {
    is_set_upper_bound(s, sup) and
    forall(b: Real) {
        is_set_upper_bound(s, b) implies sup <= b
    }
}

/// True if inf is the infimum (greatest lower bound) of the set s.
define is_set_infimum(s: Set[Real], inf: Real) -> Bool {
    is_set_lower_bound(s, inf) and
    forall(b: Real) {
        is_set_lower_bound(s, b) implies b <= inf
    }
}

/// A supremum is an upper bound for its set.
theorem set_supremum_is_upper_bound(s: Set[Real], sup: Real) {
    is_set_supremum(s, sup) implies is_set_upper_bound(s, sup)
}

/// A member of a set is below any upper bound of that set.
theorem set_upper_bound_contains_le(s: Set[Real], bound: Real, x: Real) {
    is_set_upper_bound(s, bound) and s.contains(x) implies x <= bound
}

/// A supremum is below every upper bound of its set.
theorem set_supremum_le_upper_bound(s: Set[Real], sup: Real, bound: Real) {
    is_set_supremum(s, sup) and is_set_upper_bound(s, bound) implies sup <= bound
}

/// A member of a set is below the supremum of that set.
theorem set_member_le_supremum(s: Set[Real], sup: Real, x: Real) {
    is_set_supremum(s, sup) and s.contains(x) implies x <= sup
} by {
    if is_set_supremum(s, sup) and s.contains(x) {
        set_supremum_is_upper_bound(s, sup)
        is_set_upper_bound(s, sup)
        set_upper_bound_contains_le(s, sup, x)
        x <= sup
    }
}

/// A bound strictly below the supremum is not an upper bound.
theorem set_bound_below_supremum_not_upper(s: Set[Real], sup: Real, bound: Real) {
    is_set_supremum(s, sup) and bound < sup implies not is_set_upper_bound(s, bound)
} by {
    if is_set_supremum(s, sup) and bound < sup {
        if is_set_upper_bound(s, bound) {
            set_supremum_le_upper_bound(s, sup, bound)
            sup <= bound
            bound < sup
            false
        }
    }
}

/// A number that is not an upper bound is exceeded by a member of the set.
theorem set_not_upper_bound_witness(s: Set[Real], bound: Real) {
    not is_set_upper_bound(s, bound) implies exists(x: Real) {
        s.contains(x) and not x <= bound
    }
} by {
    if not is_set_upper_bound(s, bound) {
        is_set_upper_bound(s, bound) = forall(x: Real) {
            s.contains(x) implies x <= bound
        }
        let x: Real satisfy {
            s.contains(x) and not x <= bound
        }
        exists(w: Real) {
            s.contains(w) and not w <= bound
        }
    }
}

/// A supremum is approached from below by points of the set.
theorem set_supremum_close_from_below(s: Set[Real], sup: Real, eps: Real) {
    is_set_supremum(s, sup) and eps.is_positive implies exists(x: Real) {
        s.contains(x) and sup - eps < x and x <= sup and x.is_close(sup, eps)
    }
} by {
    if is_set_supremum(s, sup) and eps.is_positive {
        let bound = sup - eps
        bound + eps = sup - eps + eps
        sup - eps + eps = sup
        bound + eps = sup
        lt_add_pos(bound, eps)
        bound < bound + eps
        bound < sup
        set_bound_below_supremum_not_upper(s, sup, bound)
        set_not_upper_bound_witness(s, bound)
        let x: Real satisfy {
            s.contains(x) and not x <= bound
        }
        s.contains(x)
        not_lte_imp_gt[Real](x, bound)
        x > bound
        bound < x
        bound = sup - eps
        sup - eps < x
        set_member_le_supremum(s, sup, x)
        x <= sup
        lt_add_pos(sup, eps)
        sup < sup + eps
        lt_of_lte_of_lt[Real](x, sup, sup + eps)
        x < sup + eps
        bounds_imp_close(x, sup, eps)
        x.is_close(sup, eps)
        s.contains(x) and sup - eps < x and x <= sup and x.is_close(sup, eps)
        exists(w: Real) {
            s.contains(w) and sup - eps < w and w <= sup and w.is_close(sup, eps)
        }
    }
}

/// Every nonempty set of real numbers that is bounded above has a supremum.
/// This is the completeness property of the real numbers.
theorem completeness(s: Set[Real]) {
    is_nonempty(s) and has_upper_bound(s)
    implies
    exists(sup: Real) {
        is_set_supremum(s, sup)
    }
} by {
    let ub: Real satisfy {
        is_set_upper_bound(s, ub)
    }

    // Convert Set[Real] to Real -> Bool representation
    let f = s.contains

    // f is nonempty
    let x: Real satisfy {
        s.contains(x)
    }
    f(x)

    // ub is an upper bound for f
    forall(y: Real) {
        if f(y) {
            y <= ub
        }
    }
    ub.is_set_upper_bound(f)

    // Apply the completeness theorem from real_set.ac
    let lub: Real satisfy {
        lub.is_set_least_upper_bound(f)
    }

    // Show that lub is the supremum of s
    // Part 1: lub is an upper bound of s
    forall(y: Real) {
        if s.contains(y) {
            f(y)
            y <= lub
        }
    }
    is_set_upper_bound(s, lub)

    // Part 2: lub is the least upper bound
    forall(b: Real) {
        if is_set_upper_bound(s, b) {
            // Show that b is an upper bound for f
            forall(y: Real) {
                if f(y) {
                    s.contains(y)
                    y <= b
                }
            }
            b.is_set_upper_bound(f)

            // Therefore lub <= b
            lub <= b
        }
    }

    is_set_supremum(s, lub)
}

/// If S ⊆ T, S is nonempty, and sup T exists, then sup S exists.
theorem supremum_exists_for_subset(s: Set[Real], t: Set[Real], sup_t: Real) {
    s.subset(t) and is_nonempty(s) and is_set_supremum(t, sup_t)
    implies
    exists(sup_s: Real) {
        is_set_supremum(s, sup_s)
    }
} by {
    // sup_t is an upper bound for s
    forall(x: Real) {
        if s.contains(x) {
            t.contains(x)
            x <= sup_t
        }
    }
    is_set_upper_bound(s, sup_t)
    has_upper_bound(s)

    // By completeness, s has a supremum
    let sup_s: Real satisfy {
        is_set_supremum(s, sup_s)
    }
}

/// If S ⊆ T, then sup S ≤ sup T.
theorem supremum_subset_le(s: Set[Real], t: Set[Real], sup_s: Real, sup_t: Real) {
    s.subset(t) and is_set_supremum(s, sup_s) and is_set_supremum(t, sup_t)
    implies
    sup_s <= sup_t
} by {
    // sup_t is an upper bound for t, hence for s
    forall(x: Real) {
        if s.contains(x) {
            t.contains(x)
            x <= sup_t
        }
    }
    is_set_upper_bound(s, sup_t)

    // sup_s is the least upper bound of s
    sup_s <= sup_t
}

/// If S and T have suprema, then sup(S ∪ T) = max(sup S, sup T).
theorem supremum_union_max(s: Set[Real], t: Set[Real], sup_s: Real, sup_t: Real) {
    is_set_supremum(s, sup_s) and is_set_supremum(t, sup_t)
    implies
    is_set_supremum(s.union(t), sup_s.max(sup_t))
} by {
    let m = sup_s.max(sup_t)

    // Part 1: m is an upper bound for s ∪ t
    forall(x: Real) {
        if s.union(t).contains(x) {
            // x is in s or in t
            if s.contains(x) {
                x <= sup_s
                m >= sup_s
                sup_s <= m
                x <= m
            } else {
                t.contains(x)
                x <= sup_t
                m >= sup_t
                sup_t <= m
                x <= m
            }
        }
    }
    is_set_upper_bound(s.union(t), m)

    // Part 2: m is the least upper bound for s ∪ t
    forall(b: Real) {
        if is_set_upper_bound(s.union(t), b) {
            // b is an upper bound for s ∪ t, so it's an upper bound for s
            forall(x: Real) {
                if s.contains(x) {
                    s.union(t).contains(x)
                    x <= b
                }
            }
            is_set_upper_bound(s, b)

            // Since sup_s is the least upper bound of s
            sup_s <= b

            // Similarly, b is an upper bound for t
            forall(x: Real) {
                if t.contains(x) {
                    s.union(t).contains(x)
                    x <= b
                }
            }
            is_set_upper_bound(t, b)

            // Since sup_t is the least upper bound of t
            sup_t <= b

            // Therefore m = max(sup_s, sup_t) ≤ b
            // By case analysis: m equals either sup_s or sup_t
            if m = sup_s {
                m <= b
            } else {
                m = sup_t
                m <= b
            }
        }
    }

    is_set_supremum(s.union(t), m)
}

/// The setwise addition of two sets S and T is the set { s+t : s∈S, t∈T }.
define setwise_add_contains(s: Set[Real], t: Set[Real], x: Real) -> Bool {
    exists(a: Real, b: Real) {
        s.contains(a) and t.contains(b) and x = a + b
    }
}

/// Returns the setwise addition of two sets.
define setwise_add(s: Set[Real], t: Set[Real]) -> Set[Real] {
    Set[Real].new(setwise_add_contains(s, t))
}

/// Membership in setwise_add is equivalent to setwise_add_contains.
theorem setwise_add_contains_iff(s: Set[Real], t: Set[Real], x: Real) {
    setwise_add(s, t).contains(x) = setwise_add_contains(s, t, x)
}

/// If a is in s and b is in t, then a + b is in their setwise sum.
theorem setwise_add_membership(s: Set[Real], t: Set[Real], a: Real, b: Real) {
    s.contains(a) and t.contains(b)
    implies
    setwise_add(s, t).contains(a + b)
} by {
    if s.contains(a) and t.contains(b) {
        setwise_add(s, t).contains(a + b) = setwise_add_contains(s, t, a + b)
        a + b = a + b
        exists(a0: Real, b0: Real) {
            s.contains(a0) and t.contains(b0) and a + b = a0 + b0
        }
        setwise_add_contains(s, t, a + b)
        setwise_add(s, t).contains(a + b)
    }
}

/// If S and T have suprema, then sup(S + T) = sup S + sup T.
theorem supremum_setwise_add(s: Set[Real], t: Set[Real], sup_s: Real, sup_t: Real) {
    is_set_supremum(s, sup_s) and is_set_supremum(t, sup_t)
    implies
    is_set_supremum(setwise_add(s, t), sup_s + sup_t)
} by {
    let sum_set = setwise_add(s, t)
    let sum_sup = sup_s + sup_t

    // Part 1: sum_sup is an upper bound for sum_set
    forall(x: Real) {
        if sum_set.contains(x) {
            // x is in the setwise sum, so x = a + b for some a ∈ s, b ∈ t
            setwise_add_contains(s, t, x)
            let (a: Real, b: Real) satisfy {
                s.contains(a) and t.contains(b) and x = a + b
            }

            // Since sup_s is an upper bound for s, we have a ≤ sup_s
            a <= sup_s

            // Since sup_t is an upper bound for t, we have b ≤ sup_t
            b <= sup_t

            // Therefore x = a + b ≤ sup_s + sup_t
            a + b <= sup_s + sup_t
            x = a + b
            x <= sum_sup
        }
    }
    is_set_upper_bound(sum_set, sum_sup)

    // Part 2: sum_sup is the least upper bound for sum_set
    forall(ub: Real) {
        if is_set_upper_bound(sum_set, ub) {
            // Show that sup_s + sup_t ≤ ub
            // Strategy: show that for all a ∈ s, a ≤ ub - sup_t
            forall(a: Real) {
                if s.contains(a) {
                    // For any t_elem ∈ t, we have a + t_elem ∈ sum_set
                    // Therefore a + t_elem ≤ ub
                    forall(t_elem: Real) {
                        if t.contains(t_elem) {
                            sum_set.contains(a + t_elem) = setwise_add(s, t).contains(a + t_elem)
                            sum_set.contains(a + t_elem)
                            a + t_elem <= ub
                            t_elem + a <= ub
                            t_elem <= ub - a
                        }
                    }

                    // Since t_elem ≤ ub - a for all t_elem ∈ t,
                    // and sup_t is the least upper bound of t,
                    // we have sup_t ≤ ub - a
                    exists(t0: Real) {
                        t.contains(t0) and not t0 <= ub - a
                    } or is_set_upper_bound(t, ub - a)
                    is_set_upper_bound(t, ub - a)
                    sup_t <= ub - a

                    // Therefore a ≤ ub - sup_t
                    sup_t + a <= ub - a + a
                    ub - a + a = ub
                    sup_t + a <= ub
                    a + sup_t <= ub
                    a <= ub - sup_t
                }
            }

            // Since a ≤ ub - sup_t for all a ∈ s, and sup_s is the least upper bound,
            // we have sup_s ≤ ub - sup_t
            exists(s0: Real) {
                s.contains(s0) and not s0 <= ub - sup_t
            } or is_set_upper_bound(s, ub - sup_t)
            is_set_upper_bound(s, ub - sup_t)
            sup_s <= ub - sup_t

            // Therefore sup_s + sup_t ≤ ub
            sum_sup <= ub
        }
    }

    is_set_supremum(sum_set, sum_sup)
}

/// The negation of a set S is the set { -x : x ∈ S }.
define negate_set_contains(s: Set[Real], x: Real) -> Bool {
    exists(a: Real) {
        s.contains(a) and x = -a
    }
}

/// The negation of a set S.
define negate_set(s: Set[Real]) -> Set[Real] {
    Set[Real].new(negate_set_contains(s))
}

/// The supremum of S equals the negative of the infimum of -S.
theorem supremum_infimum_duality(s: Set[Real], sup_s: Real) {
    is_set_supremum(s, sup_s)
    implies
    is_set_infimum(negate_set(s), -sup_s)
} by {
    let neg_s = negate_set(s)
    let inf_neg = -sup_s

    // Part 1: inf_neg is a lower bound for neg_s
    forall(x: Real) {
        if neg_s.contains(x) {
            // x is in -s, so x = -a for some a ∈ s
            negate_set_contains(s, x)
            let a: Real satisfy {
                s.contains(a) and x = -a
            }

            // Since sup_s is an upper bound for s, we have a ≤ sup_s
            a <= sup_s

            // Therefore x = -a ≥ -sup_s
            -a >= -sup_s
            x >= inf_neg
            inf_neg <= x
        }
    }
    is_set_lower_bound(neg_s, inf_neg)

    // Part 2: inf_neg is the greatest lower bound for neg_s
    forall(lb: Real) {
        if is_set_lower_bound(neg_s, lb) {
            // Show that lb ≤ -sup_s, i.e., sup_s ≤ -lb
            // Strategy: show that -lb is an upper bound for s
            forall(a: Real) {
                if s.contains(a) {
                    // -a is in neg_s
                    neg_s.contains(-a)
                    // Since lb is a lower bound for neg_s
                    -a >= lb
                    // Therefore a ≤ -lb
                    a <= -lb
                }
            }

            // -lb is an upper bound for s
            is_set_upper_bound(s, -lb)

            // Since sup_s is the least upper bound
            sup_s <= -lb

            // Therefore lb ≤ -sup_s
            lb <= -sup_s
        }
    }

    is_set_infimum(neg_s, inf_neg)
}

/// True if x is in the scalar multiple of s by alpha.
define scalar_contains(alpha: Real, s: Set[Real], x: Real) -> Bool {
    exists(a: Real) {
        s.contains(a) and x = alpha * a
    }
}

/// The scalar multiplication of a set S by a scalar α is the set { α·s : s∈S }.
define scalar_mult(alpha: Real, s: Set[Real]) -> Set[Real] {
    Set[Real].new(scalar_contains(alpha, s))
}

/// Membership in scalar_mult is equivalent to scalar_contains.
theorem scalar_mult_contains_iff(alpha: Real, s: Set[Real], x: Real) {
    scalar_mult(alpha, s).contains(x) = scalar_contains(alpha, s, x)
}

/// If x is in scalar_mult(alpha, s), then there exists a witness a in s.
theorem scalar_mult_witness(alpha: Real, s: Set[Real], x: Real) {
    scalar_mult(alpha, s).contains(x)
    implies
    exists(a: Real) {
        s.contains(a) and x = alpha * a
    }
} by {
    scalar_mult(alpha, s).contains(x) = scalar_contains(alpha, s, x)
}

/// If a is in s, then alpha * a is in scalar_mult(alpha, s).
theorem scalar_mult_membership(alpha: Real, s: Set[Real], a: Real) {
    s.contains(a)
    implies
    scalar_mult(alpha, s).contains(alpha * a)
} by {
    scalar_mult(alpha, s).contains(alpha * a) = scalar_contains(alpha, s, alpha * a)
}

/// If alpha >= 0, sup_s is the supremum of s, and for all a in s we have alpha * a <= ub,
/// then alpha * sup_s <= ub.
theorem mul_supremum_le(alpha: Real, s: Set[Real], sup_s: Real, ub: Real) {
    alpha >= Real.0 and is_set_supremum(s, sup_s) and is_nonempty(s) and
    (forall(a: Real) { s.contains(a) implies alpha * a <= ub })
    implies
    alpha * sup_s <= ub
} by {
    if alpha = Real.0 {
        // When α = 0, α·sup_s = 0
        alpha * sup_s = Real.0

        // We need to show 0 <= ub
        // Since s is nonempty, pick any element a from s
        let a: Real satisfy { s.contains(a) }
        alpha * a <= ub
        Real.0 <= ub
        alpha * sup_s <= ub
    } else {
        // When α > 0
        alpha > Real.0
        alpha != Real.0

        // For all a ∈ s, we have a ≤ ub/α
        forall(a: Real) {
            if s.contains(a) {
                alpha * a <= ub
                // Apply div_le_of_mul_le
                a <= ub / alpha
            }
        }

        // So ub/α is an upper bound of s
        exists(s0: Real) {
            s.contains(s0) and not s0 <= ub / alpha
        } or is_set_upper_bound(s, ub / alpha)
        is_set_upper_bound(s, ub / alpha)

        // Since sup_s is the least upper bound
        sup_s <= ub / alpha

        // Multiply both sides by α
        alpha * sup_s <= alpha * (ub / alpha)

        // Simplify: α * (ub/α) = ub
        alpha * (ub / alpha) = ub

        alpha * sup_s <= ub
    }
}

/// For α ≥ 0, sup(αS) = α·sup S.
theorem supremum_scalar_mult(alpha: Real, s: Set[Real], sup_s: Real) {
    alpha >= Real.0 and is_nonempty(s) and is_set_supremum(s, sup_s)
    implies
    is_set_supremum(scalar_mult(alpha, s), alpha * sup_s)
} by {
    let scaled_set = scalar_mult(alpha, s)
    let scaled_sup = alpha * sup_s

    // Part 1: scaled_sup is an upper bound for scaled_set
    forall(x: Real) {
        if scaled_set.contains(x) {
            // Apply the witness lemma explicitly
            let a: Real satisfy {
                s.contains(a) and x = alpha * a
            }

            // Since sup_s is an upper bound for s, we have a ≤ sup_s
            a <= sup_s

            // Since α ≥ 0 and a ≤ sup_s, we have α·a ≤ α·sup_s
            alpha >= Real.0
            alpha * a <= alpha * sup_s

            // Therefore x ≤ scaled_sup
            x = alpha * a
            x <= alpha * sup_s
            x <= scaled_sup
        }
    }
    is_set_upper_bound(scaled_set, scaled_sup)

    // Part 2: scaled_sup is the least upper bound for scaled_set
    forall(ub: Real) {
        if is_set_upper_bound(scaled_set, ub) {
            // Show that α·sup_s ≤ ub
            // For any a ∈ s, α·a is in scaled_set, so α·a ≤ ub
            forall(a: Real) {
                if s.contains(a) {
                    scaled_set.contains(alpha * a)
                    alpha * a <= ub
                }
            }
            forall(a0: Real) {
                s.contains(a0) implies alpha * a0 <= ub
            }

            // Apply the helper lemma
            is_nonempty(s)
            alpha >= Real.0 and is_set_supremum(s, sup_s) and is_nonempty(s) and forall(a0: Real) {
                s.contains(a0) implies alpha * a0 <= ub
            }
            mul_supremum_le(alpha, s, sup_s, ub)
            alpha * sup_s <= ub
            scaled_sup <= ub
        }
    }

    is_set_supremum(scaled_set, scaled_sup)
}

/// If beta <= 0, s is nonempty, inf_s is the infimum of s, and for all a in s
/// we have beta * a <= ub, then beta * inf_s <= ub.
theorem mul_infimum_le(beta: Real, s: Set[Real], inf_s: Real, ub: Real) {
    beta <= Real.0 and is_set_infimum(s, inf_s) and is_nonempty(s) and
    (forall(a: Real) { s.contains(a) implies beta * a <= ub })
    implies
    beta * inf_s <= ub
} by {
    if beta = Real.0 {
        // When β = 0, β·inf_s = 0
        beta * inf_s = Real.0

        // We need to show 0 <= ub
        // Since s is nonempty, pick any element a from s
        let a: Real satisfy { s.contains(a) }
        beta * a <= ub
        Real.0 <= ub
        beta * inf_s <= ub
    } else {
        // When β < 0
        beta < Real.0
        beta != Real.0
        beta.is_negative

        // For all a ∈ s, we have β·a ≤ ub
        // Dividing by β (which is negative) flips the inequality: a ≥ ub/β
        forall(a: Real) {
            if s.contains(a) {
                beta * a <= ub
                // Since β < 0, dividing by β flips the inequality
                ub / beta <= a
            }
        }

        // So ub/β is a lower bound of s
        is_set_lower_bound(s, ub / beta)

        // Since inf_s is the greatest lower bound
        ub / beta <= inf_s

        // Multiply both sides by β (which is negative), flipping the inequality
        // β·inf_s ≤ β·(ub/β) = ub
        beta * inf_s <= beta * (ub / beta)
        beta * (ub / beta) = ub
        beta * inf_s <= ub
    }
}

/// For α < 0, sup(αS) = α·inf S.
theorem supremum_scalar_mult_neg(alpha: Real, s: Set[Real], inf_s: Real) {
    alpha < Real.0 and is_nonempty(s) and is_set_infimum(s, inf_s)
    implies
    is_set_supremum(scalar_mult(alpha, s), alpha * inf_s)
} by {
    let scaled_set = scalar_mult(alpha, s)
    let scaled_sup = alpha * inf_s

    // Part 1: scaled_sup is an upper bound for scaled_set
    forall(x: Real) {
        if scaled_set.contains(x) {
            // x is in the scaled set, so x = α·a for some a ∈ s
            let a: Real satisfy {
                s.contains(a) and x = alpha * a
            }

            // Since inf_s is a lower bound for s, we have inf_s ≤ a
            inf_s <= a

            // Since α < 0 and inf_s ≤ a, multiplying by α flips the inequality
            // So α·a ≤ α·inf_s
            alpha.is_negative
            alpha * a <= alpha * inf_s
            x <= scaled_sup
        }
    }
    is_set_upper_bound(scaled_set, scaled_sup)

    // Part 2: scaled_sup is the least upper bound for scaled_set
    forall(ub: Real) {
        if is_set_upper_bound(scaled_set, ub) {
            // Show that α·inf_s ≤ ub
            // For any a ∈ s, α·a is in scaled_set, so α·a ≤ ub
            forall(a: Real) {
                if s.contains(a) {
                    scaled_set.contains(alpha * a)
                    alpha * a <= ub
                }
            }

            // Apply the helper lemma
            is_nonempty(s)
            alpha <= Real.0
            alpha * inf_s <= ub
            scaled_sup <= ub
        }
    }

    is_set_supremum(scaled_set, scaled_sup)
}

/// Convert the image of a sequence to a set.
define seq_image(a: Nat -> Real) -> Set[Real] {
    set_image(Set[Nat].universal_set, a)
}

/// Membership in `seq_image(a)` is equivalent to being a value of the sequence.
theorem seq_image_contains_iff(a: Nat -> Real, x: Real) {
    seq_image(a).contains(x) = image(a, x)
} by {
    if seq_image(a).contains(x) {
        set_image_contains_witness(Set[Nat].universal_set, a, x)
        let n: Nat satisfy {
            Set[Nat].universal_set.contains(n) and x = a(n)
        }
        image(a, x)
    }
    if image(a, x) {
        let n: Nat satisfy {
            a(n) = x
        }
        universal_set_contains_eq[Nat](n)
        Set[Nat].universal_set.contains(n)
        maps_into_set_image(Set[Nat].universal_set, a, n)
        seq_image(a).contains(a(n))
        seq_image(a).contains(x)
    }
    seq_image(a).contains(x) = image(a, x)
}

/// If an increasing sequence is bounded above, then it converges to the
/// supremum of its image. This is the monotone convergence theorem for sequences.
theorem monotone_convergence_seq(a: Nat -> Real, b: Real) {
    is_increasing(a) and is_upper_bound(a, b)
    implies
    is_set_supremum(seq_image(a), limit(a))
} by {
    // By monotone convergence principle, the sequence converges
    converges(a)

    let img = seq_image(a)
    let lim = limit(a)

    // Part 1: lim is an upper bound for img
    forall(x: Real) {
        if img.contains(x) {
            // x is in the image, so x = a(n) for some n
            seq_image_contains_iff(a, x)
            image(a, x)
            let n: Nat satisfy {
                a(n) = x
            }

            // Since the sequence converges and is increasing,
            // we have a(n) <= limit(a)
            is_increasing(a)
            converges(a)
            a(n) <= lim
            x <= lim
        }
    }
    is_set_upper_bound(img, lim)

    // Part 2: lim is the least upper bound for img
    forall(ub: Real) {
        if is_set_upper_bound(img, ub) {
            // Show that lim <= ub
            // Since ub is an upper bound for img, we have a(n) <= ub for all n
            forall(n: Nat) {
                image(a, a(n))
                seq_image_contains_iff(a, a(n))
                img.contains(a(n))
                a(n) <= ub
            }

            // So ub is an upper bound for the sequence
            is_upper_bound(a, ub)

            // Since the sequence converges to lim and is increasing,
            // we must have lim <= ub
            forall(i: Nat) {
                if Nat.0 <= i {
                    a(i) <= ub
                }
            }
            exists(n0: Nat) {
                forall(i: Nat) {
                    n0 <= i implies a(i) <= ub
                }
            }
            eventual_ub(a, ub)
            converges(a) and eventual_ub(a, ub)
            limit(a) <= ub
            lim = limit(a)
            lim <= ub
        }
    }

    is_set_supremum(img, lim)
}

/// True if f is nondecreasing (order-preserving).
define is_nondecreasing(f: Real -> Real) -> Bool {
    forall(x: Real, y: Real) {
        x <= y implies f(x) <= f(y)
    }
}

/// A monotone real function is nondecreasing.
theorem monotone_is_nondecreasing(f: Real -> Real) {
    is_monotone(f) implies is_nondecreasing(f)
} by {
    if is_monotone(f) {
        forall(x: Real, y: Real) {
            if x <= y {
                f(x) <= f(y)
            }
        }
        is_nondecreasing(f)
    }
}

/// True if y is in the image of set s under function f.
define function_image_contains(f: Real -> Real, s: Set[Real], y: Real) -> Bool {
    image_contains(f, s, y)
}

/// The image of a set s under function f.
define function_image(f: Real -> Real, s: Set[Real]) -> Set[Real] {
    set_image(s, f)
}

/// Membership in function_image is equivalent to function_image_contains.
theorem function_image_contains_iff(f: Real -> Real, s: Set[Real], y: Real) {
    function_image(f, s).contains(y) = function_image_contains(f, s, y)
}

/// Function image under composition is iterated function image.
theorem function_image_compose(f: Real -> Real, g: Real -> Real, s: Set[Real]) {
    function_image(f, function_image(g, s)) = function_image(compose(f, g), s)
} by {
    set_image_compose(s, g, f)
}

/// If f is nondecreasing, then sup f(S) ≤ f(sup S).
theorem supremum_order_preserving(f: Real -> Real, s: Set[Real], sup_s: Real) {
    is_nondecreasing(f) and is_set_supremum(s, sup_s) and is_nonempty(function_image(f, s))
    implies
    exists(sup_fs: Real) {
        is_set_supremum(function_image(f, s), sup_fs) and sup_fs <= f(sup_s)
    }
} by {
    let fs = function_image(f, s)

    // First, show that f(sup_s) is an upper bound for fs
    forall(y: Real) {
        if fs.contains(y) {
            // y is in f(s), so y = f(x) for some x in s
            fs.contains(y) = function_image_contains(f, s, y)
            function_image_contains(f, s, y)
            let x: Real satisfy {
                s.contains(x) and y = f(x)
            }

            // Since sup_s is the supremum of s, it is an upper bound
            // Therefore x <= sup_s
            // Since f is nondecreasing and x <= sup_s, f(x) <= f(sup_s)
            // Therefore y <= f(sup_s)
            y <= f(sup_s)
        }
    }
    is_set_upper_bound(fs, f(sup_s))
    has_upper_bound(fs)

    // By completeness, fs has a supremum
    let sup_fs: Real satisfy {
        is_set_supremum(fs, sup_fs)
    }

    // Since f(sup_s) is an upper bound for fs and sup_fs is the least upper bound
    sup_fs <= f(sup_s)
}

/// If f is monotone, then sup f(S) is at most f(sup S).
theorem supremum_monotone_image_le(f: Real -> Real, s: Set[Real], sup_s: Real) {
    is_monotone(f) and is_set_supremum(s, sup_s) and is_nonempty(function_image(f, s))
    implies
    exists(sup_fs: Real) {
        is_set_supremum(function_image(f, s), sup_fs) and sup_fs <= f(sup_s)
    }
} by {
    if is_monotone(f) and is_set_supremum(s, sup_s) and is_nonempty(function_image(f, s)) {
        is_nondecreasing(f)
        supremum_order_preserving(f, s, sup_s)
    }
}
