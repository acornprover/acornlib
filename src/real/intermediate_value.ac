from order import lte_refl, lte_trans, lt_trans, lt_imp_lte, not_lt_imp_gte, lte_antisymm
from order_set import closed_interval_set, closed_interval_set_contains_eq,
    closed_interval_set_lower_le, closed_interval_set_le_upper
from order import closed_interval
from real.continuity_base import Real, continuous
from real.real_base import bounds_imp_close, lt_add_pos, lt_add_right
from real.continuity_order import continuous_lt_target_neighborhood, continuous_target_lt_neighborhood,
    lt_target_neighborhood_apply, target_lt_neighborhood_apply
from real.real_seq import eps_smaller_than_both
from real.supremum import completeness, has_upper_bound, is_nonempty, is_set_supremum,
    is_set_upper_bound, set_member_le_supremum, set_supremum_close_from_below,
    set_supremum_le_upper_bound
from data.basic.set import Set

numerals Real

/// The closed-interval sublevel set used in the supremum proof of IVT.
define closed_interval_sublevel_contains(f: Real -> Real, lower: Real, upper: Real, target: Real, x: Real) -> Bool {
    lower <= x and x <= upper and f(x) <= target
}

/// The set of points in `[lower, upper]` at which `f` is at most `target`.
define closed_interval_sublevel(f: Real -> Real, lower: Real, upper: Real, target: Real) -> Set[Real] {
    Set[Real].new(closed_interval_sublevel_contains(f, lower, upper, target))
}

/// Membership in the closed-interval sublevel set is the defining conjunction.
theorem closed_interval_sublevel_contains_eq(
    f: Real -> Real, lower: Real, upper: Real, target: Real, x: Real
) {
    closed_interval_sublevel(f, lower, upper, target).contains(x) =
        closed_interval_sublevel_contains(f, lower, upper, target, x)
}

/// The left endpoint belongs to the IVT sublevel set when it is in the interval and below the target.
theorem closed_interval_sublevel_contains_lower(
    f: Real -> Real, lower: Real, upper: Real, target: Real
) {
    lower <= upper and f(lower) <= target implies
    closed_interval_sublevel(f, lower, upper, target).contains(lower)
} by {
    if lower <= upper and f(lower) <= target {
        lte_refl(lower)
        closed_interval_sublevel_contains(f, lower, upper, target, lower)
        closed_interval_sublevel_contains_eq(f, lower, upper, target, lower)
        closed_interval_sublevel(f, lower, upper, target).contains(lower)
    }
}

/// The IVT sublevel set is nonempty under the usual lower-endpoint hypothesis.
theorem closed_interval_sublevel_is_nonempty(
    f: Real -> Real, lower: Real, upper: Real, target: Real
) {
    lower <= upper and f(lower) <= target implies
    is_nonempty(closed_interval_sublevel(f, lower, upper, target))
} by {
    if lower <= upper and f(lower) <= target {
        closed_interval_sublevel_contains_lower(f, lower, upper, target)
        let s = closed_interval_sublevel(f, lower, upper, target)
        s.contains(lower)
        exists(x: Real) {
            s.contains(x)
        }
    }
}

/// The right endpoint is an upper bound for the IVT sublevel set.
theorem closed_interval_sublevel_upper_bound(
    f: Real -> Real, lower: Real, upper: Real, target: Real
) {
    is_set_upper_bound(closed_interval_sublevel(f, lower, upper, target), upper)
} by {
    let s = closed_interval_sublevel(f, lower, upper, target)
    forall(x: Real) {
        if s.contains(x) {
            closed_interval_sublevel_contains_eq(f, lower, upper, target, x)
            closed_interval_sublevel_contains(f, lower, upper, target, x)
            closed_interval_sublevel_contains(f, lower, upper, target, x) =
                (lower <= x and x <= upper and f(x) <= target)
            x <= upper
        }
    }
    function(s0: Set[Real], bound: Real) {
        forall(x0: Real) {
            s0.contains(x0) implies x0 <= bound
        } = is_set_upper_bound(s0, bound)
    }(s, upper)
    is_set_upper_bound(s, upper)
}

/// The IVT sublevel set has an upper bound.
theorem closed_interval_sublevel_has_upper_bound(
    f: Real -> Real, lower: Real, upper: Real, target: Real
) {
    has_upper_bound(closed_interval_sublevel(f, lower, upper, target))
} by {
    closed_interval_sublevel_upper_bound(f, lower, upper, target)
    let s = closed_interval_sublevel(f, lower, upper, target)
    exists(b: Real) {
        is_set_upper_bound(s, b)
    }
}

/// A supremum of the IVT sublevel set lies below the right endpoint.
theorem closed_interval_sublevel_sup_le_upper(
    f: Real -> Real, lower: Real, upper: Real, target: Real, sup: Real
) {
    is_set_supremum(closed_interval_sublevel(f, lower, upper, target), sup) implies sup <= upper
} by {
    if is_set_supremum(closed_interval_sublevel(f, lower, upper, target), sup) {
        let s = closed_interval_sublevel(f, lower, upper, target)
        closed_interval_sublevel_upper_bound(f, lower, upper, target)
        is_set_upper_bound(s, upper)
        set_supremum_le_upper_bound(s, sup, upper)
        sup <= upper
    }
}

/// A supremum of a nonempty IVT sublevel set is at least the left endpoint.
theorem closed_interval_sublevel_lower_le_sup(
    f: Real -> Real, lower: Real, upper: Real, target: Real, sup: Real
) {
    lower <= upper and f(lower) <= target and
    is_set_supremum(closed_interval_sublevel(f, lower, upper, target), sup)
    implies lower <= sup
} by {
    if lower <= upper and f(lower) <= target and
       is_set_supremum(closed_interval_sublevel(f, lower, upper, target), sup) {
        closed_interval_sublevel_contains_lower(f, lower, upper, target)
        let s = closed_interval_sublevel(f, lower, upper, target)
        s.contains(lower)
        is_set_supremum(s, sup)
        set_member_le_supremum(s, sup, lower)
        lower <= sup
    }
}

/// A supremum of the IVT sublevel set belongs to the ambient closed interval.
theorem closed_interval_sublevel_sup_in_interval(
    f: Real -> Real, lower: Real, upper: Real, target: Real, sup: Real
) {
    lower <= upper and f(lower) <= target and
    is_set_supremum(closed_interval_sublevel(f, lower, upper, target), sup)
    implies closed_interval_set(lower, upper).contains(sup)
} by {
    if lower <= upper and f(lower) <= target and
       is_set_supremum(closed_interval_sublevel(f, lower, upper, target), sup) {
        closed_interval_sublevel_lower_le_sup(f, lower, upper, target, sup)
        lower <= sup
        closed_interval_sublevel_sup_le_upper(f, lower, upper, target, sup)
        sup <= upper
        closed_interval(lower, upper, sup)
        closed_interval_set_contains_eq(lower, upper, sup)
        closed_interval_set(lower, upper).contains(sup)
    }
}

/// The IVT sublevel set has a supremum under the usual lower-endpoint hypothesis.
theorem closed_interval_sublevel_supremum_exists(
    f: Real -> Real, lower: Real, upper: Real, target: Real
) {
    lower <= upper and f(lower) <= target implies exists(sup: Real) {
        is_set_supremum(closed_interval_sublevel(f, lower, upper, target), sup)
    }
} by {
    if lower <= upper and f(lower) <= target {
        let s = closed_interval_sublevel(f, lower, upper, target)
        closed_interval_sublevel_is_nonempty(f, lower, upper, target)
        is_nonempty(s)
        closed_interval_sublevel_has_upper_bound(f, lower, upper, target)
        has_upper_bound(s)
        completeness(s)
        let sup: Real satisfy {
            is_set_supremum(s, sup)
        }
        is_set_supremum(closed_interval_sublevel(f, lower, upper, target), sup)
        exists(sup2: Real) {
            is_set_supremum(closed_interval_sublevel(f, lower, upper, target), sup2)
        }
    }
}

/// A continuous real function on a closed interval takes every intermediate value.
theorem intermediate_value_closed_interval(
    f: Real -> Real, lower: Real, upper: Real, target: Real
) {
    continuous(f) and lower <= upper and f(lower) <= target and target <= f(upper)
    implies exists(point: Real) {
        closed_interval_set(lower, upper).contains(point) and f(point) = target
    }
} by {
    if continuous(f) and lower <= upper and f(lower) <= target and target <= f(upper) {
        closed_interval_sublevel_is_nonempty(f, lower, upper, target)
        is_nonempty(closed_interval_sublevel(f, lower, upper, target))
        closed_interval_sublevel_has_upper_bound(f, lower, upper, target)
        has_upper_bound(closed_interval_sublevel(f, lower, upper, target))
        completeness(closed_interval_sublevel(f, lower, upper, target))
        exists(sup0: Real) {
            is_set_supremum(closed_interval_sublevel(f, lower, upper, target), sup0)
        }
        let sup: Real satisfy {
            is_set_supremum(closed_interval_sublevel(f, lower, upper, target), sup)
        }
        let s = closed_interval_sublevel(f, lower, upper, target)
        is_set_supremum(s, sup)
        closed_interval_sublevel_sup_in_interval(f, lower, upper, target, sup)
        closed_interval_set(lower, upper).contains(sup)
        closed_interval_set_lower_le(lower, upper, sup)
        lower <= sup
        closed_interval_set_le_upper(lower, upper, sup)
        sup <= upper
        if f(sup) < target {
            if not sup < upper {
                not_lt_imp_gte[Real](sup, upper)
                sup >= upper
                upper <= sup
                lte_antisymm[Real](sup, upper)
                sup = upper
                f(sup) = f(upper)
                target <= f(sup)
                f(sup) < target
                false
            }
            sup < upper
            let gap = upper - sup
            gap.is_positive
            continuous_lt_target_neighborhood(f, sup, target)
            let delta: Real satisfy {
                delta.is_positive and forall(y: Real) {
                    y.is_close(sup, delta) implies f(y) < target
                }
            }
            eps_smaller_than_both(delta, gap)
            let eta: Real satisfy {
                eta.is_positive and eta < delta and eta < gap
            }
            eta.is_positive
            let y = sup + eta
            lt_add_pos(sup, eta)
            sup < y

            lt_add_pos(sup - delta, delta)
            sup - delta < sup - delta + delta
            sup - delta + delta = sup
            sup - delta < sup
            lt_trans[Real](sup - delta, sup, y)
            sup - delta < y
            lt_add_right(eta, delta, sup)
            eta + sup < delta + sup
            y = eta + sup
            delta + sup = sup + delta
            y < sup + delta
            bounds_imp_close(y, sup, delta)
            y.is_close(sup, delta)
            lt_target_neighborhood_apply(f, sup, target, delta, y)
            f(y) < target
            lt_imp_lte(f(y), target)
            f(y) <= target

            lt_add_right(eta, gap, sup)
            eta + sup < gap + sup
            gap + sup = upper - sup + sup
            upper - sup + sup = upper
            y < upper
            lt_imp_lte(y, upper)
            y <= upper
            lt_imp_lte(sup, y)
            sup <= y
            lte_trans[Real](lower, sup, y)
            lower <= y
            closed_interval_sublevel_contains(f, lower, upper, target, y)
            closed_interval_sublevel_contains_eq(f, lower, upper, target, y)
            s.contains(y)
            set_member_le_supremum(s, sup, y)
            y <= sup
            sup < y
            false
        }

        if target < f(sup) {
            continuous_target_lt_neighborhood(f, sup, target)
            let delta: Real satisfy {
                delta.is_positive and forall(y: Real) {
                    y.is_close(sup, delta) implies target < f(y)
                }
            }
            set_supremum_close_from_below(s, sup, delta)
            let x: Real satisfy {
                s.contains(x) and sup - delta < x and x <= sup and x.is_close(sup, delta)
            }
            s.contains(x)
            x.is_close(sup, delta)
            target_lt_neighborhood_apply(f, sup, target, delta, x)
            target < f(x)
            closed_interval_sublevel_contains_eq(f, lower, upper, target, x)
            closed_interval_sublevel_contains(f, lower, upper, target, x)
            closed_interval_sublevel_contains(f, lower, upper, target, x) =
                (lower <= x and x <= upper and f(x) <= target)
            f(x) <= target
            false
        }

        not_lt_imp_gte[Real](f(sup), target)
        f(sup) >= target
        target <= f(sup)
        not_lt_imp_gte[Real](target, f(sup))
        target >= f(sup)
        f(sup) <= target
        lte_antisymm[Real](f(sup), target)
        f(sup) = target
        exists(point: Real) {
            closed_interval_set(lower, upper).contains(point) and f(point) = target
        }
    }
}
