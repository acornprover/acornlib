from real.real_field import Real
from real.real_series import seq_lte, partial_seq_lte, tail, partial_tail
from real.supremum import is_set_infimum, is_set_supremum, is_nonempty, is_set_upper_bound, is_set_lower_bound, has_upper_bound, function_image, function_image_contains, negate_set, negate_set_contains, completeness, set_supremum_le_upper_bound, set_member_le_supremum, set_supremum_is_upper_bound, set_upper_bound_contains_le
from data.basic.set import Set, maps_into_set_image
from data.basic.functions import function_extensionality
from algebra.semigroup import mul_fn
from algebra.add_ordered_group import add_le_add, add_le_add_right
from ordered_field import mul_le_mul_of_nonneg_right
from list import partial, partial_pointwise_eq, partial_scalar_mul
from nat import Nat, only_zero_lte_zero, lt_imp_lte_suc, lte_add_right,
    lte_add_left, lt_add_left, lte_imp_not_lt, not_lt_zero,
    add_imp_sub_left, add_sub, sub_self, lt_or_lte, lt_not_symm, lt_and_lte, lte_and_lt,
    lte_cancel_suc, lt_suc, sub_lt, alt_induction
from real.real_base import add_comm, add_assoc, add_zero_right, add_zero_left,
    add_neg_eq_zero, sub_cancels
from order import lte_antisymm, lte_trans, lt_of_lte_of_ne, not_lte_imp_gt, lt_imp_lte

numerals Real
numerals Nat

// This file lays the foundations of the Riemann (Darboux) integral for real
// functions on a closed interval.  A partition of [a, b] is given by a
// sequence p: Nat -> Real together with a length n such that p(0) = a,
// p(n) = b, and p is nondecreasing.  The lower and upper Darboux sums are
// built from the infimum and supremum of f over each subinterval.

/// True if x lies in the closed interval [a, b].
define interval_contains(a: Real, b: Real, x: Real) -> Bool {
    a <= x and x <= b
}

/// The closed interval [a, b] as a set of reals.
define interval_set(a: Real, b: Real) -> Set[Real] {
    Set[Real].new(interval_contains(a, b))
}

/// The image of the interval [x, y] under f.
define interval_image(f: Real -> Real, x: Real, y: Real) -> Set[Real] {
    function_image(f, interval_set(x, y))
}

/// True if s has a lower bound.
define has_lower_bound(s: Set[Real]) -> Bool {
    exists(b: Real) {
        is_set_lower_bound(s, b)
    }
}

/// True if the function f is bounded on the interval [a, b].
define is_bounded_on(f: Real -> Real, a: Real, b: Real) -> Bool {
    exists(lb: Real, ub: Real) {
        forall(x: Real) {
            interval_contains(a, b, x) implies lb <= f(x) and f(x) <= ub
        }
    }
}

/// A supremum of a set is unique.
theorem set_supremum_unique(s: Set[Real], m1: Real, m2: Real) {
    is_set_supremum(s, m1) and is_set_supremum(s, m2) implies m1 = m2
} by {
    if is_set_supremum(s, m1) and is_set_supremum(s, m2) {
        set_supremum_is_upper_bound(s, m2)
        is_set_upper_bound(s, m2)
        set_supremum_le_upper_bound(s, m1, m2)
        m1 <= m2
        set_supremum_is_upper_bound(s, m1)
        is_set_upper_bound(s, m1)
        set_supremum_le_upper_bound(s, m2, m1)
        m2 <= m1
        lte_antisymm[Real](m1, m2)
        m1 = m2
    }
}

/// An infimum is a lower bound for its set.
theorem set_infimum_is_lower_bound(s: Set[Real], inf: Real) {
    is_set_infimum(s, inf) implies is_set_lower_bound(s, inf)
}

/// A lower bound of a set is below the infimum of that set.
theorem set_lower_bound_le_infimum(s: Set[Real], inf: Real, lb: Real) {
    is_set_infimum(s, inf) and is_set_lower_bound(s, lb) implies lb <= inf
} by {
    if is_set_infimum(s, inf) and is_set_lower_bound(s, lb) {
        is_set_infimum(s, inf) = is_set_lower_bound(s, inf) and forall(b: Real) {
            is_set_lower_bound(s, b) implies b <= inf
        }
        is_set_lower_bound(s, inf) and forall(b: Real) {
            is_set_lower_bound(s, b) implies b <= inf
        }
        forall(b: Real) {
            is_set_lower_bound(s, b) implies b <= inf
        }
        is_set_lower_bound(s, lb) implies lb <= inf
        lb <= inf
    }
}

/// A member of a set is above any lower bound of that set.
theorem set_lower_bound_contains_le(s: Set[Real], lb: Real, x: Real) {
    is_set_lower_bound(s, lb) and s.contains(x) implies lb <= x
} by {
    if is_set_lower_bound(s, lb) and s.contains(x) {
        is_set_lower_bound(s, lb) = forall(y: Real) {
            s.contains(y) implies lb <= y
        }
        forall(y: Real) {
            s.contains(y) implies lb <= y
        }
        s.contains(x) implies lb <= x
        lb <= x
    }
}

/// Negating an inequality reverses it.
theorem neg_lte_flip(a: Real, b: Real) {
    a <= b implies -b <= -a
}

/// An infimum of a set is unique.
theorem set_infimum_unique(s: Set[Real], m1: Real, m2: Real) {
    is_set_infimum(s, m1) and is_set_infimum(s, m2) implies m1 = m2
} by {
    if is_set_infimum(s, m1) and is_set_infimum(s, m2) {
        set_infimum_is_lower_bound(s, m1)
        is_set_lower_bound(s, m1)
        set_infimum_is_lower_bound(s, m2)
        is_set_lower_bound(s, m2)
        set_lower_bound_le_infimum(s, m1, m2)
        m2 <= m1
        set_lower_bound_le_infimum(s, m2, m1)
        m1 <= m2
        lte_antisymm[Real](m1, m2)
        m1 = m2
    }
}

/// If a <= b, then a is in the interval [a, b].
theorem interval_set_contains_left(a: Real, b: Real) {
    a <= b implies interval_set(a, b).contains(a)
} by {
    if a <= b {
        interval_set(a, b).contains(a) = interval_contains(a, b, a)
        a <= a
        a <= b
        interval_contains(a, b, a)
        interval_set(a, b).contains(a)
    }
}

/// If a <= b, then b is in the interval [a, b].
theorem interval_set_contains_right(a: Real, b: Real) {
    a <= b implies interval_set(a, b).contains(b)
} by {
    if a <= b {
        interval_set(a, b).contains(b) = interval_contains(a, b, b)
        b <= b
        a <= b
        interval_contains(a, b, b)
        interval_set(a, b).contains(b)
    }
}

/// The image of a nonempty set under any function is nonempty.
theorem image_nonempty(f: Real -> Real, s: Set[Real]) {
    is_nonempty(s) implies is_nonempty(function_image(f, s))
} by {
    if is_nonempty(s) {
        let x: Real satisfy {
            s.contains(x)
        }
        maps_into_set_image(s, f, x)
        function_image(f, s).contains(f(x))
        exists(y: Real) {
            function_image(f, s).contains(y)
        }
        is_nonempty(function_image(f, s))
    }
}

/// A lower bound lb of s makes -lb an upper bound of the negation of s.
theorem neg_upper_bound_of_lower(s: Set[Real], lb: Real) {
    is_set_lower_bound(s, lb)
    implies
    is_set_upper_bound(negate_set(s), -lb)
} by {
    if is_set_lower_bound(s, lb) {
        forall(x: Real) {
            if negate_set(s).contains(x) {
                negate_set(s).contains(x) = negate_set_contains(s, x)
                negate_set_contains(s, x)
                let a: Real satisfy {
                    s.contains(a) and x = -a
                }
                set_lower_bound_contains_le(s, lb, a)
                lb <= a
                -a <= -lb
                x = -a
                x <= -lb
            }
        }
        is_set_upper_bound(negate_set(s), -lb)
    }
}

/// The negation of a nonempty set is nonempty.
theorem negate_set_nonempty(s: Set[Real]) {
    is_nonempty(s) implies is_nonempty(negate_set(s))
} by {
    if is_nonempty(s) {
        let x: Real satisfy {
            s.contains(x)
        }
        negate_set(s).contains(-x) = negate_set_contains(s, -x)
        s.contains(x) and -x = -x
        negate_set_contains(s, -x)
        negate_set(s).contains(-x)
        exists(y: Real) {
            negate_set(s).contains(y)
        }
        is_nonempty(negate_set(s))
    }
}

/// If sup is the supremum of -s, then -sup is the infimum of s.
theorem inf_of_neg_sup(s: Set[Real], sup: Real) {
    is_set_supremum(negate_set(s), sup)
    implies
    is_set_infimum(s, -sup)
} by {
    if is_set_supremum(negate_set(s), sup) {
        forall(x: Real) {
            if s.contains(x) {
                negate_set(s).contains(-x) = negate_set_contains(s, -x)
                s.contains(x) and -x = -x
                negate_set_contains(s, -x)
                negate_set(s).contains(-x)
                set_supremum_is_upper_bound(negate_set(s), sup)
                is_set_upper_bound(negate_set(s), sup)
                set_upper_bound_contains_le(negate_set(s), sup, -x)
                -x <= sup
                -sup <= x
            }
        }
        is_set_lower_bound(s, -sup)
        forall(lb: Real) {
            if is_set_lower_bound(s, lb) {
                neg_upper_bound_of_lower(s, lb)
                is_set_upper_bound(negate_set(s), -lb)
                set_supremum_le_upper_bound(negate_set(s), sup, -lb)
                sup <= -lb
                neg_lte_flip(sup, -lb)
                -(-lb) <= -sup
                lb <= -sup
            }
        }
        is_set_lower_bound(s, -sup) and forall(b: Real) {
            is_set_lower_bound(s, b) implies b <= -sup
        }
        is_set_infimum(s, -sup)
    }
}

/// If the interval is nonempty and the image has an upper bound,
/// the image has a supremum.
theorem interval_sup_exists(f: Real -> Real, x: Real, y: Real) {
    is_nonempty(interval_set(x, y)) and has_upper_bound(interval_image(f, x, y))
    implies
    exists(m: Real) {
        is_set_supremum(interval_image(f, x, y), m)
    }
} by {
    if is_nonempty(interval_set(x, y)) and has_upper_bound(interval_image(f, x, y)) {
        image_nonempty(f, interval_set(x, y))
        is_nonempty(interval_image(f, x, y))
        is_nonempty(interval_image(f, x, y)) and has_upper_bound(interval_image(f, x, y))
        completeness(interval_image(f, x, y))
        let m: Real satisfy {
            is_set_supremum(interval_image(f, x, y), m)
        }
    }
}

/// If the interval is nonempty and the image has a lower bound,
/// the image has an infimum.
theorem interval_inf_exists(f: Real -> Real, x: Real, y: Real) {
    is_nonempty(interval_set(x, y)) and has_lower_bound(interval_image(f, x, y))
    implies
    exists(m: Real) {
        is_set_infimum(interval_image(f, x, y), m)
    }
} by {
    if is_nonempty(interval_set(x, y)) and has_lower_bound(interval_image(f, x, y)) {
        image_nonempty(f, interval_set(x, y))
        is_nonempty(interval_image(f, x, y))
        negate_set_nonempty(interval_image(f, x, y))
        is_nonempty(negate_set(interval_image(f, x, y)))
        let lb: Real satisfy {
            is_set_lower_bound(interval_image(f, x, y), lb)
        }
        neg_upper_bound_of_lower(interval_image(f, x, y), lb)
        is_set_upper_bound(negate_set(interval_image(f, x, y)), -lb)
        exists(b: Real) {
            is_set_upper_bound(negate_set(interval_image(f, x, y)), b)
        }
        has_upper_bound(negate_set(interval_image(f, x, y)))
        is_nonempty(negate_set(interval_image(f, x, y))) and has_upper_bound(negate_set(interval_image(f, x, y)))
        completeness(negate_set(interval_image(f, x, y)))
        let sup: Real satisfy {
            is_set_supremum(negate_set(interval_image(f, x, y)), sup)
        }
        inf_of_neg_sup(interval_image(f, x, y), sup)
        is_set_infimum(interval_image(f, x, y), -sup)
        exists(m: Real) {
            is_set_infimum(interval_image(f, x, y), m)
        }
    }
}

/// The existence witness for the supremum on an interval, with fallback.
theorem interval_sup_exists_or_zero(f: Real -> Real, x: Real, y: Real) {
    exists(m: Real) {
        if is_nonempty(interval_set(x, y)) and has_upper_bound(interval_image(f, x, y)) {
            is_set_supremum(interval_image(f, x, y), m)
        } else {
            m = Real.0
        }
    }
} by {
    if is_nonempty(interval_set(x, y)) and has_upper_bound(interval_image(f, x, y)) {
        interval_sup_exists(f, x, y)
        let m: Real satisfy {
            is_set_supremum(interval_image(f, x, y), m)
        }
        exists(m2: Real) {
            if is_nonempty(interval_set(x, y)) and has_upper_bound(interval_image(f, x, y)) {
                is_set_supremum(interval_image(f, x, y), m2)
            } else {
                m2 = Real.0
            }
        }
    } else {
        exists(m2: Real) {
            if is_nonempty(interval_set(x, y)) and has_upper_bound(interval_image(f, x, y)) {
                is_set_supremum(interval_image(f, x, y), m2)
            } else {
                m2 = Real.0
            }
        }
    }
}

/// The existence witness for the infimum on an interval, with fallback.
theorem interval_inf_exists_or_zero(f: Real -> Real, x: Real, y: Real) {
    exists(m: Real) {
        if is_nonempty(interval_set(x, y)) and has_lower_bound(interval_image(f, x, y)) {
            is_set_infimum(interval_image(f, x, y), m)
        } else {
            m = Real.0
        }
    }
} by {
    if is_nonempty(interval_set(x, y)) and has_lower_bound(interval_image(f, x, y)) {
        interval_inf_exists(f, x, y)
        let m: Real satisfy {
            is_set_infimum(interval_image(f, x, y), m)
        }
        exists(m2: Real) {
            if is_nonempty(interval_set(x, y)) and has_lower_bound(interval_image(f, x, y)) {
                is_set_infimum(interval_image(f, x, y), m2)
            } else {
                m2 = Real.0
            }
        }
    } else {
        exists(m2: Real) {
            if is_nonempty(interval_set(x, y)) and has_lower_bound(interval_image(f, x, y)) {
                is_set_infimum(interval_image(f, x, y), m2)
            } else {
                m2 = Real.0
            }
        }
    }
}

/// The supremum of f on the interval [x, y], with fallback value zero.
let interval_sup(f: Real -> Real, x: Real, y: Real) -> m: Real satisfy {
    if is_nonempty(interval_set(x, y)) and has_upper_bound(interval_image(f, x, y)) {
        is_set_supremum(interval_image(f, x, y), m)
    } else {
        m = Real.0
    }
}

/// The infimum of f on the interval [x, y], with fallback value zero.
let interval_inf(f: Real -> Real, x: Real, y: Real) -> m: Real satisfy {
    if is_nonempty(interval_set(x, y)) and has_lower_bound(interval_image(f, x, y)) {
        is_set_infimum(interval_image(f, x, y), m)
    } else {
        m = Real.0
    }
}

/// The let-defined interval supremum satisfies its specification.
theorem interval_sup_spec(f: Real -> Real, x: Real, y: Real) {
    is_nonempty(interval_set(x, y)) and has_upper_bound(interval_image(f, x, y))
    implies
    is_set_supremum(interval_image(f, x, y), interval_sup(f, x, y))
} by {
    if is_nonempty(interval_set(x, y)) and has_upper_bound(interval_image(f, x, y)) {
        is_set_supremum(interval_image(f, x, y), interval_sup(f, x, y))
    }
}

/// The let-defined interval infimum satisfies its specification.
theorem interval_inf_spec(f: Real -> Real, x: Real, y: Real) {
    is_nonempty(interval_set(x, y)) and has_lower_bound(interval_image(f, x, y))
    implies
    is_set_infimum(interval_image(f, x, y), interval_inf(f, x, y))
} by {
    if is_nonempty(interval_set(x, y)) and has_lower_bound(interval_image(f, x, y)) {
        is_set_infimum(interval_image(f, x, y), interval_inf(f, x, y))
    }
}

/// The supremum of a function constant on an interval equals that constant.
theorem interval_sup_const(f: Real -> Real, c: Real, x: Real, y: Real) {
    x <= y and (forall(t: Real) { interval_contains(x, y, t) implies f(t) = c })
    implies
    interval_sup(f, x, y) = c
} by {
    if x <= y and (forall(t: Real) { interval_contains(x, y, t) implies f(t) = c }) {
        forall(v: Real) {
            if interval_image(f, x, y).contains(v) {
                interval_image(f, x, y).contains(v) = function_image_contains(f, interval_set(x, y), v)
                function_image_contains(f, interval_set(x, y), v)
                let t: Real satisfy {
                    interval_set(x, y).contains(t) and v = f(t)
                }
                interval_set(x, y).contains(t) = interval_contains(x, y, t)
                interval_contains(x, y, t)
                forall(t0: Real) {
                    interval_contains(x, y, t0) implies f(t0) = c
                }
                interval_contains(x, y, t) implies f(t) = c
                f(t) = c
                v = f(t)
                v = c
                v <= c
            }
        }
        is_set_upper_bound(interval_image(f, x, y), c)
        interval_set_contains_left(x, y)
        is_nonempty(interval_set(x, y))
        exists(b: Real) {
            is_set_upper_bound(interval_image(f, x, y), b)
        }
        has_upper_bound(interval_image(f, x, y))
        is_nonempty(interval_set(x, y)) and has_upper_bound(interval_image(f, x, y))
        interval_sup_spec(f, x, y)
        is_set_supremum(interval_image(f, x, y), interval_sup(f, x, y))
        set_supremum_le_upper_bound(interval_image(f, x, y), interval_sup(f, x, y), c)
        interval_sup(f, x, y) <= c
        maps_into_set_image(interval_set(x, y), f, x)
        function_image(f, interval_set(x, y)).contains(f(x))
        interval_image(f, x, y).contains(f(x))
        interval_contains(x, y, x)
        forall(t1: Real) {
            interval_contains(x, y, t1) implies f(t1) = c
        }
        interval_contains(x, y, x) implies f(x) = c
        f(x) = c
        interval_image(f, x, y).contains(c)
        set_member_le_supremum(interval_image(f, x, y), interval_sup(f, x, y), c)
        c <= interval_sup(f, x, y)
        lte_antisymm[Real](interval_sup(f, x, y), c)
        interval_sup(f, x, y) = c
    }
}

/// The infimum of a function constant on an interval equals that constant.
theorem interval_inf_const(f: Real -> Real, c: Real, x: Real, y: Real) {
    x <= y and (forall(t: Real) { interval_contains(x, y, t) implies f(t) = c })
    implies
    interval_inf(f, x, y) = c
} by {
    if x <= y and (forall(t: Real) { interval_contains(x, y, t) implies f(t) = c }) {
        forall(v: Real) {
            if interval_image(f, x, y).contains(v) {
                interval_image(f, x, y).contains(v) = function_image_contains(f, interval_set(x, y), v)
                function_image_contains(f, interval_set(x, y), v)
                let t: Real satisfy {
                    interval_set(x, y).contains(t) and v = f(t)
                }
                interval_set(x, y).contains(t) = interval_contains(x, y, t)
                interval_contains(x, y, t)
                forall(t0: Real) {
                    interval_contains(x, y, t0) implies f(t0) = c
                }
                interval_contains(x, y, t) implies f(t) = c
                f(t) = c
                v = f(t)
                v = c
                c <= v
            }
        }
        is_set_lower_bound(interval_image(f, x, y), c)
        interval_set_contains_left(x, y)
        is_nonempty(interval_set(x, y))
        exists(b: Real) {
            is_set_lower_bound(interval_image(f, x, y), b)
        }
        has_lower_bound(interval_image(f, x, y))
        is_nonempty(interval_set(x, y)) and has_lower_bound(interval_image(f, x, y))
        interval_inf_spec(f, x, y)
        is_set_infimum(interval_image(f, x, y), interval_inf(f, x, y))
        set_infimum_is_lower_bound(interval_image(f, x, y), interval_inf(f, x, y))
        is_set_lower_bound(interval_image(f, x, y), interval_inf(f, x, y))
        maps_into_set_image(interval_set(x, y), f, x)
        function_image(f, interval_set(x, y)).contains(f(x))
        interval_image(f, x, y).contains(f(x))
        interval_contains(x, y, x)
        forall(t1: Real) {
            interval_contains(x, y, t1) implies f(t1) = c
        }
        interval_contains(x, y, x) implies f(x) = c
        f(x) = c
        interval_image(f, x, y).contains(c)
        set_lower_bound_contains_le(interval_image(f, x, y), interval_inf(f, x, y), c)
        interval_inf(f, x, y) <= c
        set_lower_bound_le_infimum(interval_image(f, x, y), interval_inf(f, x, y), c)
        c <= interval_inf(f, x, y)
        lte_antisymm[Real](interval_inf(f, x, y), c)
        interval_inf(f, x, y) = c
    }
}

// ---------------------------------------------------------------------------
// Partitions and Darboux sums
// ---------------------------------------------------------------------------

/// True if p is nondecreasing up to index n.
define partition_monotone(p: Nat -> Real, n: Nat) -> Bool {
    forall(i: Nat, j: Nat) {
        i <= j and j <= n implies p(i) <= p(j)
    }
}

/// True if p is a partition of [a, b] of length n:
/// p(0) = a, p(n) = b, and p is nondecreasing.
define is_partition(p: Nat -> Real, a: Real, b: Real, n: Nat) -> Bool {
    (p(Nat.0) = a) and (p(n) = b) and partition_monotone(p, n)
}

/// The first point of a partition is a.
theorem partition_start(p: Nat -> Real, a: Real, b: Real, n: Nat) {
    is_partition(p, a, b, n) implies p(Nat.0) = a
} by {
    if is_partition(p, a, b, n) {
        is_partition(p, a, b, n) = ((p(Nat.0) = a) and (p(n) = b) and partition_monotone(p, n))
        (p(Nat.0) = a) and (p(n) = b) and partition_monotone(p, n)
        p(Nat.0) = a
    }
}

/// The last point of a partition is b.
theorem partition_end(p: Nat -> Real, a: Real, b: Real, n: Nat) {
    is_partition(p, a, b, n) implies p(n) = b
} by {
    if is_partition(p, a, b, n) {
        is_partition(p, a, b, n) = ((p(Nat.0) = a) and (p(n) = b) and partition_monotone(p, n))
        (p(Nat.0) = a) and (p(n) = b) and partition_monotone(p, n)
        p(n) = b
    }
}

/// A partition is nondecreasing on [0, n].
theorem partition_mono(p: Nat -> Real, a: Real, b: Real, n: Nat, i: Nat, j: Nat) {
    is_partition(p, a, b, n) and i <= j and j <= n implies p(i) <= p(j)
} by {
    if is_partition(p, a, b, n) and i <= j and j <= n {
        is_partition(p, a, b, n) = ((p(Nat.0) = a) and (p(n) = b) and partition_monotone(p, n))
        (p(Nat.0) = a) and (p(n) = b) and partition_monotone(p, n)
        partition_monotone(p, n)
        partition_monotone(p, n) = forall(k: Nat, l: Nat) {
            k <= l and l <= n implies p(k) <= p(l)
        }
        forall(k: Nat, l: Nat) {
            k <= l and l <= n implies p(k) <= p(l)
        }
        i <= j and j <= n implies p(i) <= p(j)
        p(i) <= p(j)
    }
}

/// Every point of a partition lies in [a, b].
theorem partition_point_in_interval(p: Nat -> Real, a: Real, b: Real, n: Nat, i: Nat) {
    is_partition(p, a, b, n) and i <= n implies interval_contains(a, b, p(i))
} by {
    if is_partition(p, a, b, n) and i <= n {
        partition_start(p, a, b, n)
        p(Nat.0) = a
        partition_mono(p, a, b, n, Nat.0, i)
        p(Nat.0) <= p(i)
        a <= p(i)
        partition_end(p, a, b, n)
        p(n) = b
        partition_mono(p, a, b, n, i, n)
        p(i) <= p(n)
        p(i) <= b
        interval_contains(a, b, p(i))
    }
}

/// The two-point partition of [a, b].
define trivial_partition(a: Real, b: Real, i: Nat) -> Real {
    if i = Nat.0 {
        a
    } else {
        b
    }
}

/// The trivial partition starts at a.
theorem trivial_partition_zero(a: Real, b: Real) {
    trivial_partition(a, b, Nat.0) = a
}

/// The trivial partition ends at b.
theorem trivial_partition_one(a: Real, b: Real) {
    trivial_partition(a, b, Nat.1) = b
}

/// The trivial partition is a partition of [a, b].
theorem trivial_partition_is_partition(a: Real, b: Real) {
    a <= b implies is_partition(trivial_partition(a, b), a, b, Nat.1)
} by {
    if a <= b {
        trivial_partition(a, b, Nat.0) = a
        trivial_partition(a, b, Nat.1) = b
        forall(i: Nat, j: Nat) {
            if i <= j and j <= Nat.1 {
                if j = Nat.0 {
                    i <= Nat.0
                    only_zero_lte_zero(i)
                    i = Nat.0
                    trivial_partition(a, b, i) = a
                    trivial_partition(a, b, j) = a
                    a <= a
                    trivial_partition(a, b, i) <= trivial_partition(a, b, j)
                } else {
                    trivial_partition(a, b, j) = b
                    if i = Nat.0 {
                        trivial_partition(a, b, i) = a
                        a <= b
                        trivial_partition(a, b, i) <= trivial_partition(a, b, j)
                    } else {
                        trivial_partition(a, b, i) = b
                        b <= b
                        trivial_partition(a, b, i) <= trivial_partition(a, b, j)
                    }
                }
            }
        }
        partition_monotone(trivial_partition(a, b), Nat.1)
        is_partition(trivial_partition(a, b), a, b, Nat.1)
    }
}

/// The successive-difference sequence of p.
define diff_step(p: Nat -> Real, i: Nat) -> Real {
    p(i + 1) - p(i)
}

/// Ring helper: a - b + c = c + a - b.
theorem ring_comm_add_sub(a: Real, b: Real, c: Real) {
    a - b + c = c + a - b
}

/// Ring helper: (a - b) + b - c = a - c.
theorem ring_cancel_sub(a: Real, b: Real, c: Real) {
    (a - b) + b - c = a - c
}

/// Ring helper for telescoping: a - b + (c - a) = c - b.
theorem ring_diff_telescope(a: Real, b: Real, c: Real) {
    a - b + (c - a) = c - b
} by {
    ring_comm_add_sub(a, b, c - a)
    a - b + (c - a) = (c - a) + a - b
    ring_cancel_sub(c, a, b)
    (c - a) + a - b = c - b
}

/// Telescoping: the sum of successive differences of p equals p(n) - p(0).
theorem telescope(p: Nat -> Real, n: Nat) {
    partial(diff_step(p), n) = p(n) - p(Nat.0)
} by {
    define q(k: Nat) -> Bool {
        partial(diff_step(p), k) = p(k) - p(Nat.0)
    }
    partial(diff_step(p), Nat.0) = Real.0
    Real.0 = p(Nat.0) - p(Nat.0)
    partial(diff_step(p), Nat.0) = p(Nat.0) - p(Nat.0)
    q(Nat.0)
    forall(k: Nat) {
        if q(k) {
            partial(diff_step(p), k.suc) = partial(diff_step(p), k) + diff_step(p, k)
            diff_step(p, k) = p(k + 1) - p(k)
            partial(diff_step(p), k) + diff_step(p, k) = p(k) - p(Nat.0) + (p(k + 1) - p(k))
            ring_diff_telescope(p(k), p(Nat.0), p(k + 1))
            p(k) - p(Nat.0) + (p(k + 1) - p(k)) = p(k + 1) - p(Nat.0)
            partial(diff_step(p), k.suc) = p(k + 1) - p(Nat.0)
            q(k.suc)
        }
    }
    q(n)
}

/// The lower Darboux step over the subinterval [x, y].
define lower_step(f: Real -> Real, x: Real, y: Real) -> Real {
    interval_inf(f, x, y) * (y - x)
}

/// The upper Darboux step over the subinterval [x, y].
define upper_step(f: Real -> Real, x: Real, y: Real) -> Real {
    interval_sup(f, x, y) * (y - x)
}

/// The lower step of a partition at index i.
define partition_step_lower(f: Real -> Real, p: Nat -> Real, i: Nat) -> Real {
    interval_inf(f, p(i), p(i + 1)) * diff_step(p, i)
}

/// The upper step of a partition at index i.
define partition_step_upper(f: Real -> Real, p: Nat -> Real, i: Nat) -> Real {
    interval_sup(f, p(i), p(i + 1)) * diff_step(p, i)
}

/// The lower Darboux sum of f over the partition p of length n.
define lower_sum(f: Real -> Real, p: Nat -> Real, n: Nat) -> Real {
    partial(partition_step_lower(f, p), n)
}

/// The upper Darboux sum of f over the partition p of length n.
define upper_sum(f: Real -> Real, p: Nat -> Real, n: Nat) -> Real {
    partial(partition_step_upper(f, p), n)
}

/// c times the successive difference of p at index i.
define scalar_diff_step(c: Real, p: Nat -> Real, i: Nat) -> Real {
    c * diff_step(p, i)
}

/// The left endpoint of a containing interval is below any member.
theorem interval_contains_left(a: Real, b: Real, x: Real) {
    interval_contains(a, b, x) implies a <= x
} by {
    if interval_contains(a, b, x) {
        interval_contains(a, b, x) = a <= x and x <= b
        a <= x and x <= b
        a <= x
    }
}

/// Any member of an interval is below the right endpoint.
theorem interval_contains_right(a: Real, b: Real, x: Real) {
    interval_contains(a, b, x) implies x <= b
} by {
    if interval_contains(a, b, x) {
        interval_contains(a, b, x) = a <= x and x <= b
        a <= x and x <= b
        x <= b
    }
}

/// A point between two points of [a, b] lies in [a, b].
theorem interval_contains_mono(a: Real, b: Real, x: Real, y: Real, t: Real) {
    interval_contains(a, b, x) and interval_contains(a, b, y) and x <= t and t <= y
    implies
    interval_contains(a, b, t)
} by {
    if interval_contains(a, b, x) and interval_contains(a, b, y) and x <= t and t <= y {
        interval_contains_left(a, b, x)
        a <= x
        interval_contains_right(a, b, y)
        y <= b
        lte_trans[Real](a, x, t)
        a <= t
        lte_trans[Real](t, y, b)
        t <= b
        interval_contains(a, b, t)
    }
}

/// The lower step of a function constant on [a, b] over a partition
/// subinterval equals c times the width of that subinterval.
theorem lower_step_const_on_partition(f: Real -> Real, p: Nat -> Real, a: Real, b: Real, n: Nat, i: Nat, c: Real) {
    is_partition(p, a, b, n) and i < n and (forall(t: Real) { interval_contains(a, b, t) implies f(t) = c })
    implies
    partition_step_lower(f, p, i) = c * diff_step(p, i)
} by {
    if is_partition(p, a, b, n) and i < n and (forall(t: Real) { interval_contains(a, b, t) implies f(t) = c }) {
        lt_imp_lte_suc(i, n)
        i + 1 <= n
        i <= i + 1
        partition_mono(p, a, b, n, i, i + 1)
        p(i) <= p(i + 1)
        partition_point_in_interval(p, a, b, n, i)
        interval_contains(a, b, p(i))
        partition_point_in_interval(p, a, b, n, i + 1)
        interval_contains(a, b, p(i + 1))
        forall(t: Real) {
            if interval_contains(p(i), p(i + 1), t) {
                interval_contains(p(i), p(i + 1), t) = p(i) <= t and t <= p(i + 1)
                p(i) <= t and t <= p(i + 1)
                p(i) <= t
                t <= p(i + 1)
                interval_contains_mono(a, b, p(i), p(i + 1), t)
                interval_contains(a, b, t)
                forall(t0: Real) {
                    interval_contains(a, b, t0) implies f(t0) = c
                }
                interval_contains(a, b, t) implies f(t) = c
                f(t) = c
            }
        }
        interval_inf_const(f, c, p(i), p(i + 1))
        interval_inf(f, p(i), p(i + 1)) = c
        partition_step_lower(f, p, i) = c * diff_step(p, i)
    }
}

/// The upper step of a function constant on [a, b] over a partition
/// subinterval equals c times the width of that subinterval.
theorem upper_step_const_on_partition(f: Real -> Real, p: Nat -> Real, a: Real, b: Real, n: Nat, i: Nat, c: Real) {
    is_partition(p, a, b, n) and i < n and (forall(t: Real) { interval_contains(a, b, t) implies f(t) = c })
    implies
    partition_step_upper(f, p, i) = c * diff_step(p, i)
} by {
    if is_partition(p, a, b, n) and i < n and (forall(t: Real) { interval_contains(a, b, t) implies f(t) = c }) {
        lt_imp_lte_suc(i, n)
        i + 1 <= n
        i <= i + 1
        partition_mono(p, a, b, n, i, i + 1)
        p(i) <= p(i + 1)
        partition_point_in_interval(p, a, b, n, i)
        interval_contains(a, b, p(i))
        partition_point_in_interval(p, a, b, n, i + 1)
        interval_contains(a, b, p(i + 1))
        forall(t: Real) {
            if interval_contains(p(i), p(i + 1), t) {
                interval_contains(p(i), p(i + 1), t) = p(i) <= t and t <= p(i + 1)
                p(i) <= t and t <= p(i + 1)
                p(i) <= t
                t <= p(i + 1)
                interval_contains_mono(a, b, p(i), p(i + 1), t)
                interval_contains(a, b, t)
                forall(t0: Real) {
                    interval_contains(a, b, t0) implies f(t0) = c
                }
                interval_contains(a, b, t) implies f(t) = c
                f(t) = c
            }
        }
        interval_sup_const(f, c, p(i), p(i + 1))
        interval_sup(f, p(i), p(i + 1)) = c
        partition_step_upper(f, p, i) = c * diff_step(p, i)
    }
}

/// c times the successive difference equals scalar multiplication by c.
theorem scalar_diff_step_eq_mul_fn(c: Real, p: Nat -> Real) {
    scalar_diff_step(c, p) = mul_fn(c, diff_step(p))
} by {
    forall(i: Nat) {
        scalar_diff_step(c, p, i) = c * diff_step(p, i)
        mul_fn(c, diff_step(p), i) = c * diff_step(p, i)
        scalar_diff_step(c, p, i) = mul_fn(c, diff_step(p), i)
    }
    function_extensionality(scalar_diff_step(c, p), mul_fn(c, diff_step(p)))
    scalar_diff_step(c, p) = mul_fn(c, diff_step(p))
}

/// The lower Darboux sum of a function constant c on [a, b] is c * (b - a)
/// for every partition.
theorem const_lower_sum(f: Real -> Real, p: Nat -> Real, a: Real, b: Real, n: Nat, c: Real) {
    is_partition(p, a, b, n) and (forall(t: Real) { interval_contains(a, b, t) implies f(t) = c })
    implies
    lower_sum(f, p, n) = c * (b - a)
} by {
    if is_partition(p, a, b, n) and (forall(t: Real) { interval_contains(a, b, t) implies f(t) = c }) {
        forall(k: Nat) {
            if k < n {
                lower_step_const_on_partition(f, p, a, b, n, k, c)
                partition_step_lower(f, p, k) = scalar_diff_step(c, p, k)
            }
        }
        partial_pointwise_eq(partition_step_lower(f, p), scalar_diff_step(c, p), n)
        partial(partition_step_lower(f, p), n) = partial(scalar_diff_step(c, p), n)
        scalar_diff_step_eq_mul_fn(c, p)
        scalar_diff_step(c, p) = mul_fn(c, diff_step(p))
        partial(scalar_diff_step(c, p), n) = partial(mul_fn(c, diff_step(p)), n)
        partial_scalar_mul(c, diff_step(p), n)
        c * partial(diff_step(p), n) = partial(mul_fn(c, diff_step(p)), n)
        partial(scalar_diff_step(c, p), n) = c * partial(diff_step(p), n)
        telescope(p, n)
        partial(diff_step(p), n) = p(n) - p(Nat.0)
        partition_start(p, a, b, n)
        p(Nat.0) = a
        partition_end(p, a, b, n)
        p(n) = b
        p(n) - p(Nat.0) = b - a
        c * (p(n) - p(Nat.0)) = c * (b - a)
        partial(partition_step_lower(f, p), n) = c * (b - a)
        lower_sum(f, p, n) = c * (b - a)
    }
}

/// The upper Darboux sum of a function constant c on [a, b] is c * (b - a)
/// for every partition.
theorem const_upper_sum(f: Real -> Real, p: Nat -> Real, a: Real, b: Real, n: Nat, c: Real) {
    is_partition(p, a, b, n) and (forall(t: Real) { interval_contains(a, b, t) implies f(t) = c })
    implies
    upper_sum(f, p, n) = c * (b - a)
} by {
    if is_partition(p, a, b, n) and (forall(t: Real) { interval_contains(a, b, t) implies f(t) = c }) {
        forall(k: Nat) {
            if k < n {
                upper_step_const_on_partition(f, p, a, b, n, k, c)
                partition_step_upper(f, p, k) = scalar_diff_step(c, p, k)
            }
        }
        partial_pointwise_eq(partition_step_upper(f, p), scalar_diff_step(c, p), n)
        partial(partition_step_upper(f, p), n) = partial(scalar_diff_step(c, p), n)
        scalar_diff_step_eq_mul_fn(c, p)
        scalar_diff_step(c, p) = mul_fn(c, diff_step(p))
        partial(scalar_diff_step(c, p), n) = partial(mul_fn(c, diff_step(p)), n)
        partial_scalar_mul(c, diff_step(p), n)
        c * partial(diff_step(p), n) = partial(mul_fn(c, diff_step(p)), n)
        partial(scalar_diff_step(c, p), n) = c * partial(diff_step(p), n)
        telescope(p, n)
        partial(diff_step(p), n) = p(n) - p(Nat.0)
        partition_start(p, a, b, n)
        p(Nat.0) = a
        partition_end(p, a, b, n)
        p(n) = b
        p(n) - p(Nat.0) = b - a
        c * (p(n) - p(Nat.0)) = c * (b - a)
        partial(partition_step_upper(f, p), n) = c * (b - a)
        upper_sum(f, p, n) = c * (b - a)
    }
}

// ---------------------------------------------------------------------------
// Integrability and the integral
// ---------------------------------------------------------------------------

/// True if s is the lower Darboux sum of f over some partition of [a, b].
define lower_sum_contains(f: Real -> Real, a: Real, b: Real, s: Real) -> Bool {
    exists(p: Nat -> Real, n: Nat) {
        is_partition(p, a, b, n) and s = lower_sum(f, p, n)
    }
}

/// The set of lower Darboux sums of f over all partitions of [a, b].
define lower_sum_set(f: Real -> Real, a: Real, b: Real) -> Set[Real] {
    Set[Real].new(lower_sum_contains(f, a, b))
}

/// True if s is the upper Darboux sum of f over some partition of [a, b].
define upper_sum_contains(f: Real -> Real, a: Real, b: Real, s: Real) -> Bool {
    exists(p: Nat -> Real, n: Nat) {
        is_partition(p, a, b, n) and s = upper_sum(f, p, n)
    }
}

/// The set of upper Darboux sums of f over all partitions of [a, b].
define upper_sum_set(f: Real -> Real, a: Real, b: Real) -> Set[Real] {
    Set[Real].new(upper_sum_contains(f, a, b))
}

/// The lower sum of a function constant on [a, b] over the trivial partition
/// is a member of its set of lower sums.
theorem const_in_lower_sum_set(f: Real -> Real, c: Real, a: Real, b: Real) {
    a <= b and (forall(t: Real) { interval_contains(a, b, t) implies f(t) = c })
    implies
    lower_sum_set(f, a, b).contains(c * (b - a))
} by {
    if a <= b and (forall(t: Real) { interval_contains(a, b, t) implies f(t) = c }) {
        trivial_partition_is_partition(a, b)
        is_partition(trivial_partition(a, b), a, b, Nat.1)
        const_lower_sum(f, trivial_partition(a, b), a, b, Nat.1, c)
        lower_sum(f, trivial_partition(a, b), Nat.1) = c * (b - a)
        c * (b - a) = lower_sum(f, trivial_partition(a, b), Nat.1)
        is_partition(trivial_partition(a, b), a, b, Nat.1) and c * (b - a) = lower_sum(f, trivial_partition(a, b), Nat.1)
        exists(p: Nat -> Real, n: Nat) {
            is_partition(p, a, b, n) and c * (b - a) = lower_sum(f, p, n)
        }
        lower_sum_contains(f, a, b, c * (b - a))
        lower_sum_set(f, a, b).contains(c * (b - a)) = lower_sum_contains(f, a, b, c * (b - a))
        lower_sum_set(f, a, b).contains(c * (b - a))
    }
}

/// The upper sum of a function constant on [a, b] over the trivial partition
/// is a member of its set of upper sums.
theorem const_in_upper_sum_set(f: Real -> Real, c: Real, a: Real, b: Real) {
    a <= b and (forall(t: Real) { interval_contains(a, b, t) implies f(t) = c })
    implies
    upper_sum_set(f, a, b).contains(c * (b - a))
} by {
    if a <= b and (forall(t: Real) { interval_contains(a, b, t) implies f(t) = c }) {
        trivial_partition_is_partition(a, b)
        is_partition(trivial_partition(a, b), a, b, Nat.1)
        const_upper_sum(f, trivial_partition(a, b), a, b, Nat.1, c)
        upper_sum(f, trivial_partition(a, b), Nat.1) = c * (b - a)
        c * (b - a) = upper_sum(f, trivial_partition(a, b), Nat.1)
        is_partition(trivial_partition(a, b), a, b, Nat.1) and c * (b - a) = upper_sum(f, trivial_partition(a, b), Nat.1)
        exists(p: Nat -> Real, n: Nat) {
            is_partition(p, a, b, n) and c * (b - a) = upper_sum(f, p, n)
        }
        upper_sum_contains(f, a, b, c * (b - a))
        upper_sum_set(f, a, b).contains(c * (b - a)) = upper_sum_contains(f, a, b, c * (b - a))
        upper_sum_set(f, a, b).contains(c * (b - a))
    }
}

/// Every lower sum of a function constant on [a, b] equals c * (b - a).
theorem lower_sum_set_all_const(f: Real -> Real, c: Real, a: Real, b: Real, s: Real) {
    (forall(t: Real) { interval_contains(a, b, t) implies f(t) = c }) and lower_sum_set(f, a, b).contains(s)
    implies
    s = c * (b - a)
} by {
    if (forall(t: Real) { interval_contains(a, b, t) implies f(t) = c }) and lower_sum_set(f, a, b).contains(s) {
        lower_sum_set(f, a, b).contains(s) = lower_sum_contains(f, a, b, s)
        lower_sum_contains(f, a, b, s)
        let (p: Nat -> Real, n: Nat) satisfy {
            is_partition(p, a, b, n) and s = lower_sum(f, p, n)
        }
        is_partition(p, a, b, n)
        const_lower_sum(f, p, a, b, n, c)
        lower_sum(f, p, n) = c * (b - a)
        s = lower_sum(f, p, n)
        s = c * (b - a)
    }
}

/// Every upper sum of a function constant on [a, b] equals c * (b - a).
theorem upper_sum_set_all_const(f: Real -> Real, c: Real, a: Real, b: Real, s: Real) {
    (forall(t: Real) { interval_contains(a, b, t) implies f(t) = c }) and upper_sum_set(f, a, b).contains(s)
    implies
    s = c * (b - a)
} by {
    if (forall(t: Real) { interval_contains(a, b, t) implies f(t) = c }) and upper_sum_set(f, a, b).contains(s) {
        upper_sum_set(f, a, b).contains(s) = upper_sum_contains(f, a, b, s)
        upper_sum_contains(f, a, b, s)
        let (p: Nat -> Real, n: Nat) satisfy {
            is_partition(p, a, b, n) and s = upper_sum(f, p, n)
        }
        is_partition(p, a, b, n)
        const_upper_sum(f, p, a, b, n, c)
        upper_sum(f, p, n) = c * (b - a)
        s = upper_sum(f, p, n)
        s = c * (b - a)
    }
}

/// If a set contains m and every member equals m, then m is its supremum.
theorem sup_of_all_equal(s: Set[Real], m: Real) {
    s.contains(m) and (forall(x: Real) { s.contains(x) implies x = m })
    implies
    is_set_supremum(s, m)
} by {
    if s.contains(m) and (forall(x: Real) { s.contains(x) implies x = m }) {
        forall(x0: Real) {
            s.contains(x0) implies x0 = m
        }
        forall(x: Real) {
            if s.contains(x) {
                forall(x1: Real) {
                    s.contains(x1) implies x1 = m
                }
                s.contains(x) implies x = m
                x = m
                x <= m
            }
        }
        is_set_upper_bound(s, m)
        forall(b: Real) {
            if is_set_upper_bound(s, b) {
                set_upper_bound_contains_le(s, b, m)
                m <= b
            }
        }
        is_set_upper_bound(s, m) and forall(b: Real) {
            is_set_upper_bound(s, b) implies m <= b
        }
        is_set_supremum(s, m)
    }
}

/// If a set contains m and every member equals m, then m is its infimum.
theorem inf_of_all_equal(s: Set[Real], m: Real) {
    s.contains(m) and (forall(x: Real) { s.contains(x) implies x = m })
    implies
    is_set_infimum(s, m)
} by {
    if s.contains(m) and (forall(x: Real) { s.contains(x) implies x = m }) {
        forall(x0: Real) {
            s.contains(x0) implies x0 = m
        }
        forall(x: Real) {
            if s.contains(x) {
                forall(x1: Real) {
                    s.contains(x1) implies x1 = m
                }
                s.contains(x) implies x = m
                x = m
                m <= x
            }
        }
        is_set_lower_bound(s, m)
        forall(b: Real) {
            if is_set_lower_bound(s, b) {
                set_lower_bound_contains_le(s, b, m)
                b <= m
            }
        }
        is_set_lower_bound(s, m) and forall(b: Real) {
            is_set_lower_bound(s, b) implies b <= m
        }
        is_set_infimum(s, m)
    }
}

/// True if f is integrable on [a, b]: the supremum of its lower sums equals
/// the infimum of its upper sums.
define is_integrable(f: Real -> Real, a: Real, b: Real) -> Bool {
    exists(m: Real) {
        is_set_supremum(lower_sum_set(f, a, b), m) and is_set_infimum(upper_sum_set(f, a, b), m)
    }
}

/// The supremum of the lower sums of a function constant on [a, b]
/// is c * (b - a).
theorem const_lower_sum_set_sup(f: Real -> Real, c: Real, a: Real, b: Real) {
    a <= b and (forall(t: Real) { interval_contains(a, b, t) implies f(t) = c })
    implies
    is_set_supremum(lower_sum_set(f, a, b), c * (b - a))
} by {
    if a <= b and (forall(t: Real) { interval_contains(a, b, t) implies f(t) = c }) {
        const_in_lower_sum_set(f, c, a, b)
        lower_sum_set(f, a, b).contains(c * (b - a))
        forall(x: Real) {
            if lower_sum_set(f, a, b).contains(x) {
                lower_sum_set_all_const(f, c, a, b, x)
                x = c * (b - a)
            }
        }
        sup_of_all_equal(lower_sum_set(f, a, b), c * (b - a))
        is_set_supremum(lower_sum_set(f, a, b), c * (b - a))
    }
}

/// The infimum of the upper sums of a function constant on [a, b]
/// is c * (b - a).
theorem const_upper_sum_set_inf(f: Real -> Real, c: Real, a: Real, b: Real) {
    a <= b and (forall(t: Real) { interval_contains(a, b, t) implies f(t) = c })
    implies
    is_set_infimum(upper_sum_set(f, a, b), c * (b - a))
} by {
    if a <= b and (forall(t: Real) { interval_contains(a, b, t) implies f(t) = c }) {
        const_in_upper_sum_set(f, c, a, b)
        upper_sum_set(f, a, b).contains(c * (b - a))
        forall(x: Real) {
            if upper_sum_set(f, a, b).contains(x) {
                upper_sum_set_all_const(f, c, a, b, x)
                x = c * (b - a)
            }
        }
        inf_of_all_equal(upper_sum_set(f, a, b), c * (b - a))
        is_set_infimum(upper_sum_set(f, a, b), c * (b - a))
    }
}

/// A function constant on [a, b] is integrable.
theorem constant_integrable(f: Real -> Real, c: Real, a: Real, b: Real) {
    a <= b and (forall(t: Real) { interval_contains(a, b, t) implies f(t) = c })
    implies
    is_integrable(f, a, b)
} by {
    if a <= b and (forall(t: Real) { interval_contains(a, b, t) implies f(t) = c }) {
        const_lower_sum_set_sup(f, c, a, b)
        is_set_supremum(lower_sum_set(f, a, b), c * (b - a))
        const_upper_sum_set_inf(f, c, a, b)
        is_set_infimum(upper_sum_set(f, a, b), c * (b - a))
        is_set_supremum(lower_sum_set(f, a, b), c * (b - a)) and is_set_infimum(upper_sum_set(f, a, b), c * (b - a))
        exists(m: Real) {
            is_set_supremum(lower_sum_set(f, a, b), m) and is_set_infimum(upper_sum_set(f, a, b), m)
        }
        is_integrable(f, a, b)
    }
}

/// The integral value exists for every f on [a, b], with fallback zero.
theorem integral_exists_or_zero(f: Real -> Real, a: Real, b: Real) {
    exists(v: Real) {
        if is_integrable(f, a, b) {
            is_set_supremum(lower_sum_set(f, a, b), v) and is_set_infimum(upper_sum_set(f, a, b), v)
        } else {
            v = Real.0
        }
    }
} by {
    if is_integrable(f, a, b) {
        let m: Real satisfy {
            is_set_supremum(lower_sum_set(f, a, b), m) and is_set_infimum(upper_sum_set(f, a, b), m)
        }
        is_set_supremum(lower_sum_set(f, a, b), m) and is_set_infimum(upper_sum_set(f, a, b), m)
        exists(v: Real) {
            if is_integrable(f, a, b) {
                is_set_supremum(lower_sum_set(f, a, b), v) and is_set_infimum(upper_sum_set(f, a, b), v)
            } else {
                v = Real.0
            }
        }
    } else {
        exists(v: Real) {
            if is_integrable(f, a, b) {
                is_set_supremum(lower_sum_set(f, a, b), v) and is_set_infimum(upper_sum_set(f, a, b), v)
            } else {
                v = Real.0
            }
        }
    }
}

/// The Riemann integral of f over [a, b]; zero when f is not integrable.
let integral(f: Real -> Real, a: Real, b: Real) -> v: Real satisfy {
    if is_integrable(f, a, b) {
        is_set_supremum(lower_sum_set(f, a, b), v) and is_set_infimum(upper_sum_set(f, a, b), v)
    } else {
        v = Real.0
    }
}

/// The let-defined integral satisfies its specification.
theorem integral_spec(f: Real -> Real, a: Real, b: Real) {
    is_integrable(f, a, b)
    implies
    is_set_supremum(lower_sum_set(f, a, b), integral(f, a, b)) and is_set_infimum(upper_sum_set(f, a, b), integral(f, a, b))
} by {
    if is_integrable(f, a, b) {
        is_set_supremum(lower_sum_set(f, a, b), integral(f, a, b)) and is_set_infimum(upper_sum_set(f, a, b), integral(f, a, b))
    }
}

/// The integral of a function constant on [a, b] is c * (b - a).
theorem integral_const(f: Real -> Real, c: Real, a: Real, b: Real) {
    a <= b and (forall(t: Real) { interval_contains(a, b, t) implies f(t) = c })
    implies
    integral(f, a, b) = c * (b - a)
} by {
    if a <= b and (forall(t: Real) { interval_contains(a, b, t) implies f(t) = c }) {
        constant_integrable(f, c, a, b)
        is_integrable(f, a, b)
        integral_spec(f, a, b)
        is_set_supremum(lower_sum_set(f, a, b), integral(f, a, b)) and is_set_infimum(upper_sum_set(f, a, b), integral(f, a, b))
        is_set_supremum(lower_sum_set(f, a, b), integral(f, a, b))
        const_lower_sum_set_sup(f, c, a, b)
        is_set_supremum(lower_sum_set(f, a, b), c * (b - a))
        set_supremum_unique(lower_sum_set(f, a, b), integral(f, a, b), c * (b - a))
        integral(f, a, b) = c * (b - a)
    }
}

// ---------------------------------------------------------------------------
// Comparison: f <= g on [a, b] implies integral(f) <= integral(g)
// ---------------------------------------------------------------------------

/// If a <= b, then b - a is nonnegative.
theorem sub_nonneg(a: Real, b: Real) {
    a <= b implies Real.0 <= b - a
} by {
    if a <= b {
        add_le_add_right[Real](a, b, -a)
        a + -a <= b + -a
        a + -a = Real.0
        b + -a = b - a
        Real.0 <= b - a
    }
}

/// The sequence f truncated to its first n terms and zero beyond.
define truncate(f: Nat -> Real, n: Nat, k: Nat) -> Real {
    if k < n {
        f(k)
    } else {
        Real.0
    }
}

/// Pointwise comparison of two sequences gives comparison of partial sums.
theorem partial_lte(f: Nat -> Real, g: Nat -> Real, n: Nat) {
    (forall(k: Nat) { k < n implies f(k) <= g(k) })
    implies
    partial(f, n) <= partial(g, n)
} by {
    if forall(k: Nat) { k < n implies f(k) <= g(k) } {
        forall(k: Nat) {
            if k < n {
                forall(k0: Nat) {
                    k0 < n implies f(k0) <= g(k0)
                }
                k < n implies f(k) <= g(k)
                f(k) <= g(k)
                truncate(f, n, k) = f(k)
                truncate(g, n, k) = g(k)
                truncate(f, n, k) <= truncate(g, n, k)
            } else {
                truncate(f, n, k) = Real.0
                truncate(g, n, k) = Real.0
                Real.0 <= Real.0
                truncate(f, n, k) <= truncate(g, n, k)
            }
        }
        seq_lte(truncate(f, n), truncate(g, n))
        partial_seq_lte(truncate(f, n), truncate(g, n))
        seq_lte(partial(truncate(f, n)), partial(truncate(g, n)))
        partial(truncate(f, n))(n) <= partial(truncate(g, n))(n)
        forall(k: Nat) {
            if k < n {
                truncate(f, n, k) = f(k)
            }
        }
        partial_pointwise_eq(truncate(f, n), f, n)
        partial(truncate(f, n), n) = partial(f, n)
        forall(k: Nat) {
            if k < n {
                truncate(g, n, k) = g(k)
            }
        }
        partial_pointwise_eq(truncate(g, n), g, n)
        partial(truncate(g, n), n) = partial(g, n)
        partial(f, n) <= partial(g, n)
    }
}

/// A pointwise lower bound on f over [x, y] makes it a lower bound of the image.
theorem image_lower_bound(f: Real -> Real, x: Real, y: Real, lb: Real) {
    (forall(t: Real) { interval_contains(x, y, t) implies lb <= f(t) })
    implies
    is_set_lower_bound(interval_image(f, x, y), lb)
} by {
    if forall(t: Real) { interval_contains(x, y, t) implies lb <= f(t) } {
        forall(v: Real) {
            if interval_image(f, x, y).contains(v) {
                interval_image(f, x, y).contains(v) = function_image_contains(f, interval_set(x, y), v)
                function_image_contains(f, interval_set(x, y), v)
                let t: Real satisfy {
                    interval_set(x, y).contains(t) and v = f(t)
                }
                interval_set(x, y).contains(t) = interval_contains(x, y, t)
                interval_contains(x, y, t)
                forall(t0: Real) {
                    interval_contains(x, y, t0) implies lb <= f(t0)
                }
                interval_contains(x, y, t) implies lb <= f(t)
                lb <= f(t)
                v = f(t)
                lb <= v
            }
        }
        is_set_lower_bound(interval_image(f, x, y), lb)
    }
}

/// If f <= g pointwise on [x, y], then the infimum of f is below that of g.
theorem interval_inf_mono(f: Real -> Real, g: Real -> Real, x: Real, y: Real, lb_f: Real, lb_g: Real) {
    x <= y and
    (forall(t: Real) { interval_contains(x, y, t) implies lb_f <= f(t) }) and
    (forall(t: Real) { interval_contains(x, y, t) implies lb_g <= g(t) }) and
    (forall(t: Real) { interval_contains(x, y, t) implies f(t) <= g(t) })
    implies
    interval_inf(f, x, y) <= interval_inf(g, x, y)
} by {
    if x <= y and
       (forall(t: Real) { interval_contains(x, y, t) implies lb_f <= f(t) }) and
       (forall(t: Real) { interval_contains(x, y, t) implies lb_g <= g(t) }) and
       (forall(t: Real) { interval_contains(x, y, t) implies f(t) <= g(t) }) {
        interval_set_contains_left(x, y)
        is_nonempty(interval_set(x, y))
        image_lower_bound(f, x, y, lb_f)
        is_set_lower_bound(interval_image(f, x, y), lb_f)
        exists(b: Real) {
            is_set_lower_bound(interval_image(f, x, y), b)
        }
        has_lower_bound(interval_image(f, x, y))
        is_nonempty(interval_set(x, y)) and has_lower_bound(interval_image(f, x, y))
        interval_inf_spec(f, x, y)
        is_set_infimum(interval_image(f, x, y), interval_inf(f, x, y))
        image_lower_bound(g, x, y, lb_g)
        is_set_lower_bound(interval_image(g, x, y), lb_g)
        exists(b0: Real) {
            is_set_lower_bound(interval_image(g, x, y), b0)
        }
        has_lower_bound(interval_image(g, x, y))
        is_nonempty(interval_set(x, y)) and has_lower_bound(interval_image(g, x, y))
        interval_inf_spec(g, x, y)
        is_set_infimum(interval_image(g, x, y), interval_inf(g, x, y))
        set_infimum_is_lower_bound(interval_image(f, x, y), interval_inf(f, x, y))
        is_set_lower_bound(interval_image(f, x, y), interval_inf(f, x, y))
        forall(v: Real) {
            if interval_image(g, x, y).contains(v) {
                interval_image(g, x, y).contains(v) = function_image_contains(g, interval_set(x, y), v)
                function_image_contains(g, interval_set(x, y), v)
                let t: Real satisfy {
                    interval_set(x, y).contains(t) and v = g(t)
                }
                interval_set(x, y).contains(t) = interval_contains(x, y, t)
                interval_contains(x, y, t)
                forall(t0: Real) {
                    interval_contains(x, y, t0) implies f(t0) <= g(t0)
                }
                interval_contains(x, y, t) implies f(t) <= g(t)
                f(t) <= g(t)
                maps_into_set_image(interval_set(x, y), f, t)
                function_image(f, interval_set(x, y)).contains(f(t))
                interval_image(f, x, y).contains(f(t))
                set_lower_bound_contains_le(interval_image(f, x, y), interval_inf(f, x, y), f(t))
                interval_inf(f, x, y) <= f(t)
                lte_trans[Real](interval_inf(f, x, y), f(t), g(t))
                interval_inf(f, x, y) <= g(t)
                v = g(t)
                interval_inf(f, x, y) <= v
            }
        }
        is_set_lower_bound(interval_image(g, x, y), interval_inf(f, x, y))
        set_lower_bound_le_infimum(interval_image(g, x, y), interval_inf(g, x, y), interval_inf(f, x, y))
        interval_inf(f, x, y) <= interval_inf(g, x, y)
    }
}

/// If f <= g pointwise on [a, b], the lower sums of f are below those of g
/// for every partition.
theorem lower_sum_mono(f: Real -> Real, g: Real -> Real, p: Nat -> Real, a: Real, b: Real, n: Nat, lb_f: Real, lb_g: Real) {
    is_partition(p, a, b, n) and
    (forall(t: Real) { interval_contains(a, b, t) implies lb_f <= f(t) }) and
    (forall(t: Real) { interval_contains(a, b, t) implies lb_g <= g(t) }) and
    (forall(t: Real) { interval_contains(a, b, t) implies f(t) <= g(t) })
    implies
    lower_sum(f, p, n) <= lower_sum(g, p, n)
} by {
    if is_partition(p, a, b, n) and
       (forall(t: Real) { interval_contains(a, b, t) implies lb_f <= f(t) }) and
       (forall(t: Real) { interval_contains(a, b, t) implies lb_g <= g(t) }) and
       (forall(t: Real) { interval_contains(a, b, t) implies f(t) <= g(t) }) {
        forall(k: Nat) {
            if k < n {
                lt_imp_lte_suc(k, n)
                k + 1 <= n
                k <= k + 1
                partition_mono(p, a, b, n, k, k + 1)
                p(k) <= p(k + 1)
                partition_point_in_interval(p, a, b, n, k)
                interval_contains(a, b, p(k))
                partition_point_in_interval(p, a, b, n, k + 1)
                interval_contains(a, b, p(k + 1))
                forall(t: Real) {
                    if interval_contains(p(k), p(k + 1), t) {
                        interval_contains_left(p(k), p(k + 1), t)
                        p(k) <= t
                        interval_contains_right(p(k), p(k + 1), t)
                        t <= p(k + 1)
                        interval_contains_mono(a, b, p(k), p(k + 1), t)
                        interval_contains(a, b, t)
                        forall(t0: Real) {
                            interval_contains(a, b, t0) implies lb_f <= f(t0)
                        }
                        interval_contains(a, b, t) implies lb_f <= f(t)
                        lb_f <= f(t)
                    }
                }
                forall(t: Real) {
                    if interval_contains(p(k), p(k + 1), t) {
                        interval_contains_left(p(k), p(k + 1), t)
                        p(k) <= t
                        interval_contains_right(p(k), p(k + 1), t)
                        t <= p(k + 1)
                        interval_contains_mono(a, b, p(k), p(k + 1), t)
                        interval_contains(a, b, t)
                        forall(t1: Real) {
                            interval_contains(a, b, t1) implies lb_g <= g(t1)
                        }
                        interval_contains(a, b, t) implies lb_g <= g(t)
                        lb_g <= g(t)
                    }
                }
                forall(t: Real) {
                    if interval_contains(p(k), p(k + 1), t) {
                        interval_contains_left(p(k), p(k + 1), t)
                        p(k) <= t
                        interval_contains_right(p(k), p(k + 1), t)
                        t <= p(k + 1)
                        interval_contains_mono(a, b, p(k), p(k + 1), t)
                        interval_contains(a, b, t)
                        forall(t2: Real) {
                            interval_contains(a, b, t2) implies f(t2) <= g(t2)
                        }
                        interval_contains(a, b, t) implies f(t) <= g(t)
                        f(t) <= g(t)
                    }
                }
                p(k) <= p(k + 1)
                forall(t: Real) {
                    interval_contains(p(k), p(k + 1), t) implies lb_f <= f(t)
                }
                forall(t: Real) {
                    interval_contains(p(k), p(k + 1), t) implies lb_g <= g(t)
                }
                forall(t: Real) {
                    interval_contains(p(k), p(k + 1), t) implies f(t) <= g(t)
                }
                interval_set_contains_left(p(k), p(k + 1))
                is_nonempty(interval_set(p(k), p(k + 1)))
                image_lower_bound(f, p(k), p(k + 1), lb_f)
                is_set_lower_bound(interval_image(f, p(k), p(k + 1)), lb_f)
                exists(b1: Real) {
                    is_set_lower_bound(interval_image(f, p(k), p(k + 1)), b1)
                }
                has_lower_bound(interval_image(f, p(k), p(k + 1)))
                is_nonempty(interval_set(p(k), p(k + 1))) and has_lower_bound(interval_image(f, p(k), p(k + 1)))
                interval_inf_spec(f, p(k), p(k + 1))
                is_set_infimum(interval_image(f, p(k), p(k + 1)), interval_inf(f, p(k), p(k + 1)))
                set_infimum_is_lower_bound(interval_image(f, p(k), p(k + 1)), interval_inf(f, p(k), p(k + 1)))
                is_set_lower_bound(interval_image(f, p(k), p(k + 1)), interval_inf(f, p(k), p(k + 1)))
                image_lower_bound(g, p(k), p(k + 1), lb_g)
                is_set_lower_bound(interval_image(g, p(k), p(k + 1)), lb_g)
                exists(b2: Real) {
                    is_set_lower_bound(interval_image(g, p(k), p(k + 1)), b2)
                }
                has_lower_bound(interval_image(g, p(k), p(k + 1)))
                is_nonempty(interval_set(p(k), p(k + 1))) and has_lower_bound(interval_image(g, p(k), p(k + 1)))
                interval_inf_spec(g, p(k), p(k + 1))
                is_set_infimum(interval_image(g, p(k), p(k + 1)), interval_inf(g, p(k), p(k + 1)))
                forall(v: Real) {
                    if interval_image(g, p(k), p(k + 1)).contains(v) {
                        interval_image(g, p(k), p(k + 1)).contains(v) = function_image_contains(g, interval_set(p(k), p(k + 1)), v)
                        function_image_contains(g, interval_set(p(k), p(k + 1)), v)
                        let t: Real satisfy {
                            interval_set(p(k), p(k + 1)).contains(t) and v = g(t)
                        }
                        interval_set(p(k), p(k + 1)).contains(t) = interval_contains(p(k), p(k + 1), t)
                        interval_contains(p(k), p(k + 1), t)
                        forall(t2: Real) {
                            interval_contains(p(k), p(k + 1), t2) implies f(t2) <= g(t2)
                        }
                        interval_contains(p(k), p(k + 1), t) implies f(t) <= g(t)
                        f(t) <= g(t)
                        maps_into_set_image(interval_set(p(k), p(k + 1)), f, t)
                        function_image(f, interval_set(p(k), p(k + 1))).contains(f(t))
                        interval_image(f, p(k), p(k + 1)).contains(f(t))
                        set_lower_bound_contains_le(interval_image(f, p(k), p(k + 1)), interval_inf(f, p(k), p(k + 1)), f(t))
                        interval_inf(f, p(k), p(k + 1)) <= f(t)
                        lte_trans[Real](interval_inf(f, p(k), p(k + 1)), f(t), g(t))
                        interval_inf(f, p(k), p(k + 1)) <= g(t)
                        v = g(t)
                        interval_inf(f, p(k), p(k + 1)) <= v
                    }
                }
                is_set_lower_bound(interval_image(g, p(k), p(k + 1)), interval_inf(f, p(k), p(k + 1)))
                set_lower_bound_le_infimum(interval_image(g, p(k), p(k + 1)), interval_inf(g, p(k), p(k + 1)), interval_inf(f, p(k), p(k + 1)))
                interval_inf(f, p(k), p(k + 1)) <= interval_inf(g, p(k), p(k + 1))
                sub_nonneg(p(k), p(k + 1))
                Real.0 <= p(k + 1) - p(k)
                mul_le_mul_of_nonneg_right(interval_inf(f, p(k), p(k + 1)), interval_inf(g, p(k), p(k + 1)), p(k + 1) - p(k))
                interval_inf(f, p(k), p(k + 1)) * (p(k + 1) - p(k)) <= interval_inf(g, p(k), p(k + 1)) * (p(k + 1) - p(k))
                partition_step_lower(f, p, k) <= partition_step_lower(g, p, k)
            }
        }
        partial_lte(partition_step_lower(f, p), partition_step_lower(g, p), n)
        partial(partition_step_lower(f, p), n) <= partial(partition_step_lower(g, p), n)
        lower_sum(f, p, n) <= lower_sum(g, p, n)
    }
}

/// The supremum of a set is at most any upper bound of that set.
theorem sup_le_of_upper_bound(s: Set[Real], sup_s: Real, b: Real) {
    is_set_supremum(s, sup_s) and (forall(x: Real) { s.contains(x) implies x <= b })
    implies
    sup_s <= b
} by {
    if is_set_supremum(s, sup_s) and (forall(x: Real) { s.contains(x) implies x <= b }) {
        forall(x0: Real) {
            s.contains(x0) implies x0 <= b
        }
        is_set_upper_bound(s, b)
        set_supremum_le_upper_bound(s, sup_s, b)
        sup_s <= b
    }
}

/// If f <= g on [a, b], the supremum of the lower sums of f is at most the
/// supremum of the lower sums of g.
theorem sup_lower_le_sup_lower(f: Real -> Real, g: Real -> Real, a: Real, b: Real, sup_f: Real, sup_g: Real, lb_f: Real, lb_g: Real) {
    a <= b and
    is_set_supremum(lower_sum_set(f, a, b), sup_f) and
    is_set_supremum(lower_sum_set(g, a, b), sup_g) and
    (forall(t: Real) { interval_contains(a, b, t) implies lb_f <= f(t) }) and
    (forall(t: Real) { interval_contains(a, b, t) implies lb_g <= g(t) }) and
    (forall(t: Real) { interval_contains(a, b, t) implies f(t) <= g(t) })
    implies
    sup_f <= sup_g
} by {
    if a <= b and
       is_set_supremum(lower_sum_set(f, a, b), sup_f) and
       is_set_supremum(lower_sum_set(g, a, b), sup_g) and
       (forall(t: Real) { interval_contains(a, b, t) implies lb_f <= f(t) }) and
       (forall(t: Real) { interval_contains(a, b, t) implies lb_g <= g(t) }) and
       (forall(t: Real) { interval_contains(a, b, t) implies f(t) <= g(t) }) {
        forall(x: Real) {
            if lower_sum_set(f, a, b).contains(x) {
                lower_sum_set(f, a, b).contains(x) = lower_sum_contains(f, a, b, x)
                lower_sum_contains(f, a, b, x)
                let (p: Nat -> Real, n: Nat) satisfy {
                    is_partition(p, a, b, n) and x = lower_sum(f, p, n)
                }
                is_partition(p, a, b, n)
                lower_sum_mono(f, g, p, a, b, n, lb_f, lb_g)
                lower_sum(f, p, n) <= lower_sum(g, p, n)
                lower_sum_contains(g, a, b, lower_sum(g, p, n))
                lower_sum_set(g, a, b).contains(lower_sum(g, p, n)) = lower_sum_contains(g, a, b, lower_sum(g, p, n))
                lower_sum_set(g, a, b).contains(lower_sum(g, p, n))
                set_member_le_supremum(lower_sum_set(g, a, b), sup_g, lower_sum(g, p, n))
                lower_sum(g, p, n) <= sup_g
                lte_trans[Real](lower_sum(f, p, n), lower_sum(g, p, n), sup_g)
                lower_sum(f, p, n) <= sup_g
                x = lower_sum(f, p, n)
                x <= sup_g
            }
        }
        sup_le_of_upper_bound(lower_sum_set(f, a, b), sup_f, sup_g)
        sup_f <= sup_g
    }
}

/// Comparison theorem: if f <= g on [a, b], both are integrable, and both are
/// bounded below by lb_f and lb_g, then the integral of f is at most the
/// integral of g.
theorem comparison(f: Real -> Real, g: Real -> Real, a: Real, b: Real, lb_f: Real, lb_g: Real) {
    a <= b and
    is_integrable(f, a, b) and is_integrable(g, a, b) and
    (forall(t: Real) { interval_contains(a, b, t) implies lb_f <= f(t) }) and
    (forall(t: Real) { interval_contains(a, b, t) implies lb_g <= g(t) }) and
    (forall(t: Real) { interval_contains(a, b, t) implies f(t) <= g(t) })
    implies
    integral(f, a, b) <= integral(g, a, b)
} by {
    if a <= b and
       is_integrable(f, a, b) and is_integrable(g, a, b) and
       (forall(t: Real) { interval_contains(a, b, t) implies lb_f <= f(t) }) and
       (forall(t: Real) { interval_contains(a, b, t) implies lb_g <= g(t) }) and
       (forall(t: Real) { interval_contains(a, b, t) implies f(t) <= g(t) }) {
        integral_spec(f, a, b)
        is_set_supremum(lower_sum_set(f, a, b), integral(f, a, b)) and is_set_infimum(upper_sum_set(f, a, b), integral(f, a, b))
        is_set_supremum(lower_sum_set(f, a, b), integral(f, a, b))
        integral_spec(g, a, b)
        is_set_supremum(lower_sum_set(g, a, b), integral(g, a, b)) and is_set_infimum(upper_sum_set(g, a, b), integral(g, a, b))
        is_set_supremum(lower_sum_set(g, a, b), integral(g, a, b))
        sup_lower_le_sup_lower(f, g, a, b, integral(f, a, b), integral(g, a, b), lb_f, lb_g)
        integral(f, a, b) <= integral(g, a, b)
    }
}

// The bounded version of the comparison theorem.  Its proof needs the lemma
// that a function bounded on [a, b] has a pointwise lower bound; that lemma is
// currently left as future work, so the theorem is stated here as a comment.
//
// theorem comparison_bounded(f: Real -> Real, g: Real -> Real, a: Real, b: Real) {
//     a <= b and
//     is_integrable(f, a, b) and is_integrable(g, a, b) and
//     is_bounded_on(f, a, b) and is_bounded_on(g, a, b) and
//     (forall(t: Real) { interval_contains(a, b, t) implies f(t) <= g(t) })
//     implies
//     integral(f, a, b) <= integral(g, a, b)
// }

// ---------------------------------------------------------------------------
// Concatenation of partitions and additivity of the integral
// ---------------------------------------------------------------------------

// Every function integrable on [a, b] is bounded on [a, b].
// The proof needs the argument that an unbounded function has unbounded
// Darboux sums over some partition, so the supremum of the lower sums cannot
// exist; that argument is left as future work.
//
// theorem integrable_imp_bounded_on(f: Real -> Real, a: Real, b: Real) {
//     is_integrable(f, a, b) implies is_bounded_on(f, a, b)
// }

// ---------------------------------------------------------------------------
// Natural number arithmetic for concatenating partitions
// ---------------------------------------------------------------------------

/// Adding a common amount to both sides of a natural inequality can be
/// cancelled.
theorem nat_lte_cancel_add_right(x: Nat, y: Nat, b: Nat) {
    x + b <= y + b implies x <= y
} by {
    define p(k: Nat) -> Bool {
        x + k <= y + k implies x <= y
    }
    x + Nat.0 = x
    y + Nat.0 = y
    if x + Nat.0 <= y + Nat.0 {
        x <= y
    }
    p(Nat.0) = (x + Nat.0 <= y + Nat.0 implies x <= y)
    p(Nat.0)
    forall(k: Nat) {
        if p(k) {
            if x + k.suc <= y + k.suc {
                lte_cancel_suc(x + k, y + k)
                x + k <= y + k
                x <= y
            }
            p(k.suc) = (x + k.suc <= y + k.suc implies x <= y)
            p(k.suc)
        }
    }
    forall(k: Nat) { p(k) implies p(k.suc) }
    alt_induction(p)
    p(Nat.0) and forall(k: Nat) { p(k) implies p(k.suc) } implies forall(k: Nat) { p(k) }
    p(Nat.0) and forall(k: Nat) { p(k) implies p(k.suc) }
    forall(k: Nat) { p(k) }
    p(b)
}

/// A bound on a sum gives a bound on the subtraction: a <= b + c implies
/// a - b <= c.
theorem nat_sub_lte(a: Nat, b: Nat, c: Nat) {
    a <= b + c implies a - b <= c
} by {
    if a <= b + c {
        if b <= a {
            add_sub(a, b)
            a - b + b = a
            a - b + b <= b + c
            nat_lte_cancel_add_right(a - b, c, b)
            a - b <= c
        } else {
            not b <= a
            not_lte_imp_gt[Nat](b, a)
            a < b
            sub_lt(a, b)
            a - b = Nat.0
            only_zero_lte_zero(c)
            Nat.0 <= c
            a - b <= c
        }
    }
}

/// Subtraction is monotone: a <= b implies a - k <= b - k.
theorem nat_sub_le_sub(a: Nat, b: Nat, k: Nat) {
    a <= b implies a - k <= b - k
} by {
    if a <= b {
        if k <= a {
            add_sub(a, k)
            a - k + k = a
            lte_trans[Nat](k, a, b)
            k <= b
            add_sub(b, k)
            b - k + k = b
            a - k + k <= b - k + k
            nat_lte_cancel_add_right(a - k, b - k, k)
            a - k <= b - k
        } else {
            not k <= a
            not_lte_imp_gt[Nat](k, a)
            a < k
            sub_lt(a, k)
            a - k = Nat.0
            only_zero_lte_zero(b - k)
            Nat.0 <= b - k
            a - k <= b - k
        }
    }
}

/// The successor distributes over subtraction when the subtracted amount does
/// not exceed the base: b <= a implies (a + 1) - b = (a - b) + 1.
theorem nat_suc_sub(a: Nat, b: Nat) {
    b <= a implies (a + 1) - b = (a - b) + 1
} by {
    if b <= a {
        add_sub(a, b)
        a - b + b = a
        (a - b) + 1 + b = a + 1
        add_imp_sub_left((a - b) + 1, b, a + 1)
        (a + 1) - b = (a - b) + 1
    }
}

/// Ring helper: (a + b) - b = a.
theorem ring_add_sub_cancel(a: Real, b: Real) {
    (a + b) - b = a
} by {
    sub_cancels(a, b)
    a + b - b = a
    (a + b) - b = a
}

/// Ring helper: (a - b) + b = a.
theorem ring_sub_add_cancel(a: Real, b: Real) {
    (a - b) + b = a
} by {
    add_comm(a - b, b)
    (a - b) + b = b + (a - b)
    a - b = a + -b
    b + (a - b) = b + (a + -b)
    add_comm(a, -b)
    a + -b = -b + a
    b + (a + -b) = b + (-b + a)
    add_assoc(b, -b, a)
    (b + -b) + a = b + (-b + a)
    add_neg_eq_zero(b)
    b + -b = Real.0
    (b + -b) + a = Real.0 + a
    add_zero_left(a)
    Real.0 + a = a
    b + (a - b) = a
    (a - b) + b = a
}

// The additivity identity integral(f, a, c) = integral(f, a, b) +
// integral(f, b, c) for a <= b <= c is proved here directly from the Darboux
// definition, without common refinements: concatenating a partition of [a, b]
// with a partition of [b, c] gives a partition of [a, c] whose lower (upper)
// Darboux sum is the sum of the two.  Hence every sum of a lower sum on [a, b]
// and a lower sum on [b, c] is a lower sum on [a, c] (and dually for upper
// sums), which squeezes the suprema and infima together.

/// The concatenation of a partition p1 of [a, b] of length n1 with a partition
/// p2 of [b, c]: the first n1 + 1 points come from p1 and the rest from p2.
define concat_partition(p1: Nat -> Real, n1: Nat, p2: Nat -> Real, i: Nat) -> Real {
    if i <= n1 {
        p1(i)
    } else {
        p2(i - n1)
    }
}

/// The concatenated partition starts at the start of p1.
theorem concat_partition_start(p1: Nat -> Real, n1: Nat, p2: Nat -> Real) {
    concat_partition(p1, n1, p2, Nat.0) = p1(Nat.0)
} by {
    concat_partition(p1, n1, p2, Nat.0) = (if Nat.0 <= n1 { p1(Nat.0) } else { p2(Nat.0 - n1) })
    only_zero_lte_zero(n1)
    Nat.0 <= n1
    (if Nat.0 <= n1 { p1(Nat.0) } else { p2(Nat.0 - n1) }) = p1(Nat.0)
    concat_partition(p1, n1, p2, Nat.0) = p1(Nat.0)
}

/// At index n1 + i the concatenated partition agrees with p2 at i.
theorem concat_partition_tail_point(p1: Nat -> Real, n1: Nat, p2: Nat -> Real, a: Real, b: Real, c: Real, n2: Nat, i: Nat) {
    is_partition(p1, a, b, n1) and is_partition(p2, b, c, n2)
    implies concat_partition(p1, n1, p2, n1 + i) = p2(i)
} by {
    if is_partition(p1, a, b, n1) and is_partition(p2, b, c, n2) {
        if i = Nat.0 {
            n1 + Nat.0 = n1
            concat_partition(p1, n1, p2, n1 + Nat.0) = concat_partition(p1, n1, p2, n1)
            concat_partition(p1, n1, p2, n1) = p1(n1)
            partition_end(p1, a, b, n1)
            p1(n1) = b
            partition_start(p2, b, c, n2)
            p2(Nat.0) = b
            p2(i) = p2(Nat.0)
            concat_partition(p1, n1, p2, n1 + i) = p2(i)
        } else {
            i != Nat.0
            only_zero_lte_zero(i)
            Nat.0 <= i
            lt_of_lte_of_ne[Nat](Nat.0, i)
            Nat.0 < i
            lt_add_left(n1, Nat.0, i)
            n1 + Nat.0 < n1 + i
            n1 < n1 + i
            lte_imp_not_lt(n1 + i, n1)
            not (n1 + i <= n1)
            concat_partition(p1, n1, p2, n1 + i) = p2((n1 + i) - n1)
            add_imp_sub_left(n1, i, n1 + i)
            (n1 + i) - n1 = i
            p2((n1 + i) - n1) = p2(i)
            concat_partition(p1, n1, p2, n1 + i) = p2(i)
        }
    }
}

/// The concatenated partition ends at the end of p2, which is c.
theorem concat_partition_end(p1: Nat -> Real, n1: Nat, p2: Nat -> Real, a: Real, b: Real, c: Real, n2: Nat) {
    is_partition(p1, a, b, n1) and is_partition(p2, b, c, n2)
    implies concat_partition(p1, n1, p2, n1 + n2) = c
} by {
    if is_partition(p1, a, b, n1) and is_partition(p2, b, c, n2) {
        concat_partition_tail_point(p1, n1, p2, a, b, c, n2, n2)
        concat_partition(p1, n1, p2, n1 + n2) = p2(n2)
        partition_end(p2, b, c, n2)
        p2(n2) = c
        concat_partition(p1, n1, p2, n1 + n2) = c
    }
}

/// A point of the concatenated partition below n1 comes from p1.
theorem concat_partition_prefix_point(p1: Nat -> Real, n1: Nat, p2: Nat -> Real, i: Nat) {
    i <= n1 implies concat_partition(p1, n1, p2, i) = p1(i)
} by {
    if i <= n1 {
        concat_partition(p1, n1, p2, i) = p1(i)
    }
}

/// A point of the concatenated partition above n1 comes from p2.
theorem concat_partition_suffix_point(p1: Nat -> Real, n1: Nat, p2: Nat -> Real, a: Real, b: Real, c: Real, n2: Nat, i: Nat) {
    is_partition(p1, a, b, n1) and is_partition(p2, b, c, n2) and n1 <= i and i < n1 + n2
    implies concat_partition(p1, n1, p2, i) = p2(i - n1)
} by {
    if is_partition(p1, a, b, n1) and is_partition(p2, b, c, n2) and n1 <= i and i < n1 + n2 {
        if i = n1 {
            concat_partition(p1, n1, p2, i) = p1(n1)
            partition_end(p1, a, b, n1)
            p1(n1) = b
            partition_start(p2, b, c, n2)
            p2(Nat.0) = b
            sub_self(n1)
            i - n1 = n1 - n1
            sub_self(n1)
            n1 - n1 = Nat.0
            i - n1 = Nat.0
            p2(i - n1) = p2(Nat.0)
            concat_partition(p1, n1, p2, i) = p2(i - n1)
        } else {
            i != n1
            lt_of_lte_of_ne[Nat](n1, i)
            n1 < i
            concat_partition(p1, n1, p2, i) = p2(i - n1)
        }
    }
}

/// The successor index of a suffix point of the concatenated partition also
/// agrees with p2 (including the final index n1 + n2).
theorem concat_partition_suffix_suc_point(p1: Nat -> Real, n1: Nat, p2: Nat -> Real, a: Real, b: Real, c: Real, n2: Nat, i: Nat) {
    is_partition(p1, a, b, n1) and is_partition(p2, b, c, n2) and n1 <= i and i < n1 + n2
    implies concat_partition(p1, n1, p2, i + 1) = p2((i + 1) - n1)
} by {
    if is_partition(p1, a, b, n1) and is_partition(p2, b, c, n2) and n1 <= i and i < n1 + n2 {
        if i + 1 = n1 + n2 {
            concat_partition_tail_point(p1, n1, p2, a, b, c, n2, n2)
            concat_partition(p1, n1, p2, n1 + n2) = p2(n2)
            add_imp_sub_left(n1, n2, n1 + n2)
            (n1 + n2) - n1 = n2
            concat_partition(p1, n1, p2, i + 1) = p2((i + 1) - n1)
        } else {
            i + 1 != n1 + n2
            lt_imp_lte_suc(i, n1 + n2)
            i + 1 <= n1 + n2
            lt_of_lte_of_ne[Nat](i + 1, n1 + n2)
            i + 1 < n1 + n2
            lt_suc(i)
            i < i + 1
            lt_imp_lte(i, i + 1)
            i <= i + 1
            lte_trans[Nat](n1, i, i + 1)
            n1 <= i + 1
            concat_partition_suffix_point(p1, n1, p2, a, b, c, n2, i + 1)
            concat_partition(p1, n1, p2, i + 1) = p2((i + 1) - n1)
        }
    }
}

/// The concatenation of two partitions is nondecreasing.
theorem concat_partition_mono(p1: Nat -> Real, p2: Nat -> Real, a: Real, b: Real, c: Real, n1: Nat, n2: Nat, i: Nat, j: Nat) {
    is_partition(p1, a, b, n1) and is_partition(p2, b, c, n2) and i <= j and j <= n1 + n2
    implies concat_partition(p1, n1, p2, i) <= concat_partition(p1, n1, p2, j)
} by {
    if is_partition(p1, a, b, n1) and is_partition(p2, b, c, n2) and i <= j and j <= n1 + n2 {
        if j <= n1 {
            lte_trans[Nat](i, j, n1)
            i <= n1
            concat_partition_prefix_point(p1, n1, p2, i)
            concat_partition(p1, n1, p2, i) = p1(i)
            concat_partition_prefix_point(p1, n1, p2, j)
            concat_partition(p1, n1, p2, j) = p1(j)
            partition_mono(p1, a, b, n1, i, j)
            p1(i) <= p1(j)
            concat_partition(p1, n1, p2, i) <= concat_partition(p1, n1, p2, j)
        } else {
            not j <= n1
            not_lte_imp_gt[Nat](j, n1)
            n1 < j
            if i <= n1 {
                concat_partition_prefix_point(p1, n1, p2, i)
                concat_partition(p1, n1, p2, i) = p1(i)
                concat_partition_suffix_point(p1, n1, p2, a, b, c, n2, j)
                concat_partition(p1, n1, p2, j) = p2(j - n1)
                partition_mono(p1, a, b, n1, i, n1)
                p1(i) <= p1(n1)
                partition_end(p1, a, b, n1)
                p1(n1) = b
                p1(i) <= b
                only_zero_lte_zero(j - n1)
                Nat.0 <= j - n1
                nat_sub_lte(j, n1, n2)
                j - n1 <= n2
                partition_mono(p2, b, c, n2, Nat.0, j - n1)
                p2(Nat.0) <= p2(j - n1)
                partition_start(p2, b, c, n2)
                p2(Nat.0) = b
                b <= p2(j - n1)
                lte_trans[Real](p1(i), b, p2(j - n1))
                p1(i) <= p2(j - n1)
                concat_partition(p1, n1, p2, i) <= concat_partition(p1, n1, p2, j)
            } else {
                not i <= n1
                not_lte_imp_gt[Nat](i, n1)
                n1 < i
                concat_partition_suffix_point(p1, n1, p2, a, b, c, n2, i)
                concat_partition(p1, n1, p2, i) = p2(i - n1)
                concat_partition_suffix_point(p1, n1, p2, a, b, c, n2, j)
                concat_partition(p1, n1, p2, j) = p2(j - n1)
                nat_sub_le_sub(i, j, n1)
                i - n1 <= j - n1
                nat_sub_lte(j, n1, n2)
                j - n1 <= n2
                partition_mono(p2, b, c, n2, i - n1, j - n1)
                p2(i - n1) <= p2(j - n1)
                concat_partition(p1, n1, p2, i) <= concat_partition(p1, n1, p2, j)
            }
        }
    }
}

/// The concatenation of a partition of [a, b] and a partition of [b, c] is a
/// partition of [a, c].
theorem concat_partition_is_partition(p1: Nat -> Real, p2: Nat -> Real, a: Real, b: Real, c: Real, n1: Nat, n2: Nat) {
    is_partition(p1, a, b, n1) and is_partition(p2, b, c, n2)
    implies is_partition(concat_partition(p1, n1, p2), a, c, n1 + n2)
} by {
    if is_partition(p1, a, b, n1) and is_partition(p2, b, c, n2) {
        concat_partition_start(p1, n1, p2)
        concat_partition(p1, n1, p2, Nat.0) = p1(Nat.0)
        partition_start(p1, a, b, n1)
        p1(Nat.0) = a
        concat_partition(p1, n1, p2, Nat.0) = a
        concat_partition_end(p1, n1, p2, a, b, c, n2)
        concat_partition(p1, n1, p2, n1 + n2) = c
        forall(i: Nat, j: Nat) {
            if i <= j and j <= n1 + n2 {
                concat_partition_mono(p1, p2, a, b, c, n1, n2, i, j)
                concat_partition(p1, n1, p2, i) <= concat_partition(p1, n1, p2, j)
            }
        }
        partition_monotone(concat_partition(p1, n1, p2), n1 + n2)
        is_partition(concat_partition(p1, n1, p2), a, c, n1 + n2)
    }
}

/// The lower Darboux step of the concatenated partition agrees with p1 on the
/// prefix indices.
theorem concat_partition_step_lower_prefix(f: Real -> Real, p1: Nat -> Real, n1: Nat, p2: Nat -> Real, i: Nat) {
    i < n1 implies partition_step_lower(f, concat_partition(p1, n1, p2), i) = partition_step_lower(f, p1, i)
} by {
    if i < n1 {
        lt_imp_lte_suc(i, n1)
        i + 1 <= n1
        i <= i + 1
        lte_trans[Nat](i, i + 1, n1)
        i <= n1
        concat_partition_prefix_point(p1, n1, p2, i)
        concat_partition(p1, n1, p2, i) = p1(i)
        concat_partition_prefix_point(p1, n1, p2, i + 1)
        concat_partition(p1, n1, p2, i + 1) = p1(i + 1)
        partition_step_lower(f, concat_partition(p1, n1, p2), i) =
            interval_inf(f, concat_partition(p1, n1, p2, i), concat_partition(p1, n1, p2, i + 1)) * diff_step(concat_partition(p1, n1, p2), i)
        interval_inf(f, concat_partition(p1, n1, p2, i), concat_partition(p1, n1, p2, i + 1)) * diff_step(concat_partition(p1, n1, p2), i) =
            interval_inf(f, p1(i), p1(i + 1)) * diff_step(p1, i)
        partition_step_lower(f, concat_partition(p1, n1, p2), i) = partition_step_lower(f, p1, i)
    }
}

/// The lower Darboux step of the concatenated partition agrees with p2 on the
/// suffix indices.
theorem concat_partition_step_lower_suffix(f: Real -> Real, p1: Nat -> Real, n1: Nat, p2: Nat -> Real, a: Real, b: Real, c: Real, n2: Nat, i: Nat) {
    is_partition(p1, a, b, n1) and is_partition(p2, b, c, n2) and n1 <= i and i < n1 + n2
    implies partition_step_lower(f, concat_partition(p1, n1, p2), i) = partition_step_lower(f, p2, i - n1)
} by {
    if is_partition(p1, a, b, n1) and is_partition(p2, b, c, n2) and n1 <= i and i < n1 + n2 {
        concat_partition_suffix_point(p1, n1, p2, a, b, c, n2, i)
        concat_partition(p1, n1, p2, i) = p2(i - n1)
        concat_partition_suffix_suc_point(p1, n1, p2, a, b, c, n2, i)
        concat_partition(p1, n1, p2, i + 1) = p2((i + 1) - n1)
        nat_suc_sub(i, n1)
        (i + 1) - n1 = (i - n1) + 1
        p2((i + 1) - n1) = p2((i - n1) + 1)
        partition_step_lower(f, concat_partition(p1, n1, p2), i) =
            interval_inf(f, concat_partition(p1, n1, p2, i), concat_partition(p1, n1, p2, i + 1)) * diff_step(concat_partition(p1, n1, p2), i)
        interval_inf(f, concat_partition(p1, n1, p2, i), concat_partition(p1, n1, p2, i + 1)) * diff_step(concat_partition(p1, n1, p2), i) =
            interval_inf(f, p2(i - n1), p2((i - n1) + 1)) * diff_step(p2, i - n1)
        partition_step_lower(f, concat_partition(p1, n1, p2), i) = partition_step_lower(f, p2, i - n1)
    }
}

/// The lower Darboux sum of the concatenated partition is the sum of the two
/// lower Darboux sums.
theorem concat_partition_lower_sum(f: Real -> Real, p1: Nat -> Real, p2: Nat -> Real, a: Real, b: Real, c: Real, n1: Nat, n2: Nat) {
    is_partition(p1, a, b, n1) and is_partition(p2, b, c, n2)
    implies lower_sum(f, concat_partition(p1, n1, p2), n1 + n2) = lower_sum(f, p1, n1) + lower_sum(f, p2, n2)
} by {
    if is_partition(p1, a, b, n1) and is_partition(p2, b, c, n2) {
        partial_tail(partition_step_lower(f, concat_partition(p1, n1, p2)), n1, n2)
        partial(partition_step_lower(f, concat_partition(p1, n1, p2)), n1 + n2) =
            partial(partition_step_lower(f, concat_partition(p1, n1, p2)), n1) +
            partial(tail(partition_step_lower(f, concat_partition(p1, n1, p2)), n1), n2)
        forall(k: Nat) {
            if k < n1 {
                concat_partition_step_lower_prefix(f, p1, n1, p2, k)
                partition_step_lower(f, concat_partition(p1, n1, p2), k) = partition_step_lower(f, p1, k)
            }
        }
        partial_pointwise_eq(partition_step_lower(f, concat_partition(p1, n1, p2)), partition_step_lower(f, p1), n1)
        partial(partition_step_lower(f, concat_partition(p1, n1, p2)), n1) = partial(partition_step_lower(f, p1), n1)
        forall(k: Nat) {
            if k < n2 {
                n1 <= n1 + k
                lt_add_left(n1, k, n2)
                n1 + k < n1 + n2
                concat_partition_step_lower_suffix(f, p1, n1, p2, a, b, c, n2, n1 + k)
                partition_step_lower(f, concat_partition(p1, n1, p2), n1 + k) = partition_step_lower(f, p2, (n1 + k) - n1)
                add_imp_sub_left(n1, k, n1 + k)
                (n1 + k) - n1 = k
                partition_step_lower(f, concat_partition(p1, n1, p2), n1 + k) = partition_step_lower(f, p2, k)
                tail(partition_step_lower(f, concat_partition(p1, n1, p2)), n1, k) =
                    partition_step_lower(f, concat_partition(p1, n1, p2), n1 + k)
                tail(partition_step_lower(f, concat_partition(p1, n1, p2)), n1, k) = partition_step_lower(f, p2, k)
            }
        }
        partial_pointwise_eq(tail(partition_step_lower(f, concat_partition(p1, n1, p2)), n1), partition_step_lower(f, p2), n2)
        partial(tail(partition_step_lower(f, concat_partition(p1, n1, p2)), n1), n2) = partial(partition_step_lower(f, p2), n2)
        partial(partition_step_lower(f, concat_partition(p1, n1, p2)), n1 + n2) =
            partial(partition_step_lower(f, p1), n1) + partial(partition_step_lower(f, p2), n2)
        lower_sum(f, concat_partition(p1, n1, p2), n1 + n2) = lower_sum(f, p1, n1) + lower_sum(f, p2, n2)
    }
}

/// The upper Darboux step of the concatenated partition agrees with p1 on the
/// prefix indices.
theorem concat_partition_step_upper_prefix(f: Real -> Real, p1: Nat -> Real, n1: Nat, p2: Nat -> Real, i: Nat) {
    i < n1 implies partition_step_upper(f, concat_partition(p1, n1, p2), i) = partition_step_upper(f, p1, i)
} by {
    if i < n1 {
        lt_imp_lte_suc(i, n1)
        i + 1 <= n1
        i <= i + 1
        lte_trans[Nat](i, i + 1, n1)
        i <= n1
        concat_partition_prefix_point(p1, n1, p2, i)
        concat_partition(p1, n1, p2, i) = p1(i)
        concat_partition_prefix_point(p1, n1, p2, i + 1)
        concat_partition(p1, n1, p2, i + 1) = p1(i + 1)
        partition_step_upper(f, concat_partition(p1, n1, p2), i) =
            interval_sup(f, concat_partition(p1, n1, p2, i), concat_partition(p1, n1, p2, i + 1)) * diff_step(concat_partition(p1, n1, p2), i)
        interval_sup(f, concat_partition(p1, n1, p2, i), concat_partition(p1, n1, p2, i + 1)) * diff_step(concat_partition(p1, n1, p2), i) =
            interval_sup(f, p1(i), p1(i + 1)) * diff_step(p1, i)
        partition_step_upper(f, concat_partition(p1, n1, p2), i) = partition_step_upper(f, p1, i)
    }
}

/// The upper Darboux step of the concatenated partition agrees with p2 on the
/// suffix indices.
theorem concat_partition_step_upper_suffix(f: Real -> Real, p1: Nat -> Real, n1: Nat, p2: Nat -> Real, a: Real, b: Real, c: Real, n2: Nat, i: Nat) {
    is_partition(p1, a, b, n1) and is_partition(p2, b, c, n2) and n1 <= i and i < n1 + n2
    implies partition_step_upper(f, concat_partition(p1, n1, p2), i) = partition_step_upper(f, p2, i - n1)
} by {
    if is_partition(p1, a, b, n1) and is_partition(p2, b, c, n2) and n1 <= i and i < n1 + n2 {
        concat_partition_suffix_point(p1, n1, p2, a, b, c, n2, i)
        concat_partition(p1, n1, p2, i) = p2(i - n1)
        concat_partition_suffix_suc_point(p1, n1, p2, a, b, c, n2, i)
        concat_partition(p1, n1, p2, i + 1) = p2((i + 1) - n1)
        nat_suc_sub(i, n1)
        (i + 1) - n1 = (i - n1) + 1
        p2((i + 1) - n1) = p2((i - n1) + 1)
        partition_step_upper(f, concat_partition(p1, n1, p2), i) =
            interval_sup(f, concat_partition(p1, n1, p2, i), concat_partition(p1, n1, p2, i + 1)) * diff_step(concat_partition(p1, n1, p2), i)
        interval_sup(f, concat_partition(p1, n1, p2, i), concat_partition(p1, n1, p2, i + 1)) * diff_step(concat_partition(p1, n1, p2), i) =
            interval_sup(f, p2(i - n1), p2((i - n1) + 1)) * diff_step(p2, i - n1)
        partition_step_upper(f, concat_partition(p1, n1, p2), i) = partition_step_upper(f, p2, i - n1)
    }
}

/// The upper Darboux sum of the concatenated partition is the sum of the two
/// upper Darboux sums.
theorem concat_partition_upper_sum(f: Real -> Real, p1: Nat -> Real, p2: Nat -> Real, a: Real, b: Real, c: Real, n1: Nat, n2: Nat) {
    is_partition(p1, a, b, n1) and is_partition(p2, b, c, n2)
    implies upper_sum(f, concat_partition(p1, n1, p2), n1 + n2) = upper_sum(f, p1, n1) + upper_sum(f, p2, n2)
} by {
    if is_partition(p1, a, b, n1) and is_partition(p2, b, c, n2) {
        partial_tail(partition_step_upper(f, concat_partition(p1, n1, p2)), n1, n2)
        partial(partition_step_upper(f, concat_partition(p1, n1, p2)), n1 + n2) =
            partial(partition_step_upper(f, concat_partition(p1, n1, p2)), n1) +
            partial(tail(partition_step_upper(f, concat_partition(p1, n1, p2)), n1), n2)
        forall(k: Nat) {
            if k < n1 {
                concat_partition_step_upper_prefix(f, p1, n1, p2, k)
                partition_step_upper(f, concat_partition(p1, n1, p2), k) = partition_step_upper(f, p1, k)
            }
        }
        partial_pointwise_eq(partition_step_upper(f, concat_partition(p1, n1, p2)), partition_step_upper(f, p1), n1)
        partial(partition_step_upper(f, concat_partition(p1, n1, p2)), n1) = partial(partition_step_upper(f, p1), n1)
        forall(k: Nat) {
            if k < n2 {
                n1 <= n1 + k
                lt_add_left(n1, k, n2)
                n1 + k < n1 + n2
                concat_partition_step_upper_suffix(f, p1, n1, p2, a, b, c, n2, n1 + k)
                partition_step_upper(f, concat_partition(p1, n1, p2), n1 + k) = partition_step_upper(f, p2, (n1 + k) - n1)
                add_imp_sub_left(n1, k, n1 + k)
                (n1 + k) - n1 = k
                partition_step_upper(f, concat_partition(p1, n1, p2), n1 + k) = partition_step_upper(f, p2, k)
                tail(partition_step_upper(f, concat_partition(p1, n1, p2)), n1, k) =
                    partition_step_upper(f, concat_partition(p1, n1, p2), n1 + k)
                tail(partition_step_upper(f, concat_partition(p1, n1, p2)), n1, k) = partition_step_upper(f, p2, k)
            }
        }
        partial_pointwise_eq(tail(partition_step_upper(f, concat_partition(p1, n1, p2)), n1), partition_step_upper(f, p2), n2)
        partial(tail(partition_step_upper(f, concat_partition(p1, n1, p2)), n1), n2) = partial(partition_step_upper(f, p2), n2)
        partial(partition_step_upper(f, concat_partition(p1, n1, p2)), n1 + n2) =
            partial(partition_step_upper(f, p1), n1) + partial(partition_step_upper(f, p2), n2)
        upper_sum(f, concat_partition(p1, n1, p2), n1 + n2) = upper_sum(f, p1, n1) + upper_sum(f, p2, n2)
    }
}

/// Every lower Darboux sum of [a, b] plus every lower Darboux sum of [b, c]
/// is a lower Darboux sum of [a, c], hence at most the supremum of the lower
/// sums on [a, c].
theorem additivity_concat_lower(f: Real -> Real, a: Real, b: Real, c: Real, m_ac: Real) {
    a <= b and b <= c and is_set_supremum(lower_sum_set(f, a, c), m_ac)
    implies forall(x1: Real, x2: Real) {
        lower_sum_set(f, a, b).contains(x1) and lower_sum_set(f, b, c).contains(x2)
        implies x1 + x2 <= m_ac
    }
} by {
    if a <= b and b <= c and is_set_supremum(lower_sum_set(f, a, c), m_ac) {
        forall(x1: Real, x2: Real) {
            if lower_sum_set(f, a, b).contains(x1) and lower_sum_set(f, b, c).contains(x2) {
                lower_sum_set(f, a, b).contains(x1) = lower_sum_contains(f, a, b, x1)
                lower_sum_contains(f, a, b, x1)
                let (p1: Nat -> Real, n1: Nat) satisfy {
                    is_partition(p1, a, b, n1) and x1 = lower_sum(f, p1, n1)
                }
                lower_sum_set(f, b, c).contains(x2) = lower_sum_contains(f, b, c, x2)
                lower_sum_contains(f, b, c, x2)
                let (p2: Nat -> Real, n2: Nat) satisfy {
                    is_partition(p2, b, c, n2) and x2 = lower_sum(f, p2, n2)
                }
                concat_partition_is_partition(p1, p2, a, b, c, n1, n2)
                is_partition(concat_partition(p1, n1, p2), a, c, n1 + n2)
                concat_partition_lower_sum(f, p1, p2, a, b, c, n1, n2)
                lower_sum(f, concat_partition(p1, n1, p2), n1 + n2) = lower_sum(f, p1, n1) + lower_sum(f, p2, n2)
                lower_sum(f, p1, n1) + lower_sum(f, p2, n2) = x1 + x2
                lower_sum(f, concat_partition(p1, n1, p2), n1 + n2) = x1 + x2
                is_partition(concat_partition(p1, n1, p2), a, c, n1 + n2) and
                    x1 + x2 = lower_sum(f, concat_partition(p1, n1, p2), n1 + n2)
                exists(p: Nat -> Real, n: Nat) {
                    is_partition(p, a, c, n) and x1 + x2 = lower_sum(f, p, n)
                }
                lower_sum_contains(f, a, c, x1 + x2)
                lower_sum_set(f, a, c).contains(x1 + x2) = lower_sum_contains(f, a, c, x1 + x2)
                lower_sum_set(f, a, c).contains(x1 + x2)
                set_member_le_supremum(lower_sum_set(f, a, c), m_ac, x1 + x2)
                x1 + x2 <= m_ac
            }
        }
    }
}

/// The supremum of the lower sums on [a, b] plus that on [b, c] is at most the
/// supremum of the lower sums on [a, c].
theorem sup_lower_add_le(f: Real -> Real, a: Real, b: Real, c: Real, m_ab: Real, m_bc: Real, m_ac: Real) {
    a <= b and b <= c and
    is_set_supremum(lower_sum_set(f, a, b), m_ab) and
    is_set_supremum(lower_sum_set(f, b, c), m_bc) and
    is_set_supremum(lower_sum_set(f, a, c), m_ac)
    implies m_ab + m_bc <= m_ac
} by {
    if a <= b and b <= c and
       is_set_supremum(lower_sum_set(f, a, b), m_ab) and
       is_set_supremum(lower_sum_set(f, b, c), m_bc) and
       is_set_supremum(lower_sum_set(f, a, c), m_ac) {
        additivity_concat_lower(f, a, b, c, m_ac)
        forall(x2: Real) {
            if lower_sum_set(f, b, c).contains(x2) {
                forall(x1: Real) {
                    if lower_sum_set(f, a, b).contains(x1) {
                        lower_sum_set(f, a, b).contains(x1) and lower_sum_set(f, b, c).contains(x2)
                        x1 + x2 <= m_ac
                        add_le_add_right[Real](x1 + x2, m_ac, -x2)
                        (x1 + x2) + -x2 <= m_ac + -x2
                        sub_cancels(x1, x2)
                        (x1 + x2) + -x2 = x1
                        x1 <= m_ac + -x2
                        m_ac + -x2 = m_ac - x2
                        x1 <= m_ac - x2
                    }
                }
                is_set_upper_bound(lower_sum_set(f, a, b), m_ac - x2)
                sup_le_of_upper_bound(lower_sum_set(f, a, b), m_ab, m_ac - x2)
                m_ab <= m_ac - x2
                add_le_add_right[Real](m_ab, m_ac - x2, x2)
                m_ab + x2 <= (m_ac - x2) + x2
                ring_sub_add_cancel(m_ac, x2)
                (m_ac - x2) + x2 = m_ac
                m_ab + x2 <= m_ac
                x2 + m_ab = m_ab + x2
                x2 + m_ab <= m_ac
                add_le_add_right[Real](x2 + m_ab, m_ac, -m_ab)
                (x2 + m_ab) + -m_ab <= m_ac + -m_ab
                sub_cancels(x2, m_ab)
                (x2 + m_ab) + -m_ab = x2
                x2 <= m_ac + -m_ab
                m_ac + -m_ab = m_ac - m_ab
                x2 <= m_ac - m_ab
            }
        }
        forall(x2: Real) {
            if lower_sum_set(f, b, c).contains(x2) {
                x2 <= m_ac - m_ab
            }
        }
        is_set_upper_bound(lower_sum_set(f, b, c), m_ac - m_ab)
        sup_le_of_upper_bound(lower_sum_set(f, b, c), m_bc, m_ac - m_ab)
        m_bc <= m_ac - m_ab
        add_le_add_right[Real](m_bc, m_ac - m_ab, m_ab)
        m_bc + m_ab <= (m_ac - m_ab) + m_ab
        (m_ac - m_ab) + m_ab = m_ac
        m_bc + m_ab <= m_ac
        m_ab + m_bc = m_bc + m_ab
        m_ab + m_bc <= m_ac
    }
}

/// Every upper Darboux sum of [a, b] plus every upper Darboux sum of [b, c]
/// is an upper Darboux sum of [a, c], hence at least the infimum of the upper
/// sums on [a, c].
theorem additivity_concat_upper(f: Real -> Real, a: Real, b: Real, c: Real, m_ac: Real) {
    a <= b and b <= c and is_set_infimum(upper_sum_set(f, a, c), m_ac)
    implies forall(x1: Real, x2: Real) {
        upper_sum_set(f, a, b).contains(x1) and upper_sum_set(f, b, c).contains(x2)
        implies m_ac <= x1 + x2
    }
} by {
    if a <= b and b <= c and is_set_infimum(upper_sum_set(f, a, c), m_ac) {
        forall(x1: Real, x2: Real) {
            if upper_sum_set(f, a, b).contains(x1) and upper_sum_set(f, b, c).contains(x2) {
                upper_sum_set(f, a, b).contains(x1) = upper_sum_contains(f, a, b, x1)
                upper_sum_contains(f, a, b, x1)
                let (p1: Nat -> Real, n1: Nat) satisfy {
                    is_partition(p1, a, b, n1) and x1 = upper_sum(f, p1, n1)
                }
                upper_sum_set(f, b, c).contains(x2) = upper_sum_contains(f, b, c, x2)
                upper_sum_contains(f, b, c, x2)
                let (p2: Nat -> Real, n2: Nat) satisfy {
                    is_partition(p2, b, c, n2) and x2 = upper_sum(f, p2, n2)
                }
                concat_partition_is_partition(p1, p2, a, b, c, n1, n2)
                is_partition(concat_partition(p1, n1, p2), a, c, n1 + n2)
                concat_partition_upper_sum(f, p1, p2, a, b, c, n1, n2)
                upper_sum(f, concat_partition(p1, n1, p2), n1 + n2) = upper_sum(f, p1, n1) + upper_sum(f, p2, n2)
                upper_sum(f, p1, n1) + upper_sum(f, p2, n2) = x1 + x2
                upper_sum(f, concat_partition(p1, n1, p2), n1 + n2) = x1 + x2
                is_partition(concat_partition(p1, n1, p2), a, c, n1 + n2) and
                    x1 + x2 = upper_sum(f, concat_partition(p1, n1, p2), n1 + n2)
                exists(p: Nat -> Real, n: Nat) {
                    is_partition(p, a, c, n) and x1 + x2 = upper_sum(f, p, n)
                }
                upper_sum_contains(f, a, c, x1 + x2)
                upper_sum_set(f, a, c).contains(x1 + x2) = upper_sum_contains(f, a, c, x1 + x2)
                upper_sum_set(f, a, c).contains(x1 + x2)
                set_infimum_is_lower_bound(upper_sum_set(f, a, c), m_ac)
                is_set_lower_bound(upper_sum_set(f, a, c), m_ac)
                set_lower_bound_contains_le(upper_sum_set(f, a, c), m_ac, x1 + x2)
                m_ac <= x1 + x2
            }
        }
    }
}

/// The infimum of the upper sums on [a, c] is at most the sum of the infima
/// of the upper sums on [a, b] and [b, c].
theorem inf_upper_add_ge(f: Real -> Real, a: Real, b: Real, c: Real, m_ab: Real, m_bc: Real, m_ac: Real) {
    a <= b and b <= c and
    is_set_infimum(upper_sum_set(f, a, b), m_ab) and
    is_set_infimum(upper_sum_set(f, b, c), m_bc) and
    is_set_infimum(upper_sum_set(f, a, c), m_ac)
    implies m_ac <= m_ab + m_bc
} by {
    if a <= b and b <= c and
       is_set_infimum(upper_sum_set(f, a, b), m_ab) and
       is_set_infimum(upper_sum_set(f, b, c), m_bc) and
       is_set_infimum(upper_sum_set(f, a, c), m_ac) {
        additivity_concat_upper(f, a, b, c, m_ac)
        forall(x2: Real) {
            if upper_sum_set(f, b, c).contains(x2) {
                forall(x1: Real) {
                    if upper_sum_set(f, a, b).contains(x1) {
                        upper_sum_set(f, a, b).contains(x1) and upper_sum_set(f, b, c).contains(x2)
                        m_ac <= x1 + x2
                        add_le_add_right[Real](m_ac, x1 + x2, -x2)
                        m_ac + -x2 <= (x1 + x2) + -x2
                        sub_cancels(x1, x2)
                        (x1 + x2) + -x2 = x1
                        m_ac + -x2 <= x1
                        m_ac + -x2 = m_ac - x2
                        m_ac - x2 <= x1
                    }
                }
                is_set_lower_bound(upper_sum_set(f, a, b), m_ac - x2)
                set_lower_bound_le_infimum(upper_sum_set(f, a, b), m_ab, m_ac - x2)
                m_ac - x2 <= m_ab
                add_le_add_right[Real](m_ac - x2, m_ab, x2)
                (m_ac - x2) + x2 <= m_ab + x2
                ring_sub_add_cancel(m_ac, x2)
                (m_ac - x2) + x2 = m_ac
                m_ac <= m_ab + x2
                m_ab + x2 = x2 + m_ab
                m_ac <= x2 + m_ab
                add_le_add_right[Real](m_ac, x2 + m_ab, -m_ab)
                m_ac + -m_ab <= (x2 + m_ab) + -m_ab
                sub_cancels(x2, m_ab)
                (x2 + m_ab) + -m_ab = x2
                m_ac + -m_ab <= x2
                m_ac + -m_ab = m_ac - m_ab
                m_ac - m_ab <= x2
            }
        }
        forall(x2: Real) {
            if upper_sum_set(f, b, c).contains(x2) {
                m_ac - m_ab <= x2
            }
        }
        is_set_lower_bound(upper_sum_set(f, b, c), m_ac - m_ab)
        set_lower_bound_le_infimum(upper_sum_set(f, b, c), m_bc, m_ac - m_ab)
        m_ac - m_ab <= m_bc
        add_le_add_right[Real](m_ac - m_ab, m_bc, m_ab)
        (m_ac - m_ab) + m_ab <= m_bc + m_ab
        ring_sub_add_cancel(m_ac, m_ab)
        (m_ac - m_ab) + m_ab = m_ac
        m_ac <= m_bc + m_ab
        m_bc + m_ab = m_ab + m_bc
        m_ac <= m_ab + m_bc
    }
}

/// Additivity of the integral over adjacent intervals: if f is integrable on
/// [a, b], [b, c], and [a, c] with a <= b <= c, then the integral over [a, c]
/// is the sum of the integrals over [a, b] and [b, c].
theorem integral_additivity(f: Real -> Real, a: Real, b: Real, c: Real) {
    a <= b and b <= c and is_integrable(f, a, b) and is_integrable(f, b, c) and is_integrable(f, a, c)
    implies integral(f, a, c) = integral(f, a, b) + integral(f, b, c)
} by {
    if a <= b and b <= c and is_integrable(f, a, b) and is_integrable(f, b, c) and is_integrable(f, a, c) {
        integral_spec(f, a, b)
        is_set_supremum(lower_sum_set(f, a, b), integral(f, a, b)) and
            is_set_infimum(upper_sum_set(f, a, b), integral(f, a, b))
        integral_spec(f, b, c)
        is_set_supremum(lower_sum_set(f, b, c), integral(f, b, c)) and
            is_set_infimum(upper_sum_set(f, b, c), integral(f, b, c))
        integral_spec(f, a, c)
        is_set_supremum(lower_sum_set(f, a, c), integral(f, a, c)) and
            is_set_infimum(upper_sum_set(f, a, c), integral(f, a, c))
        sup_lower_add_le(f, a, b, c, integral(f, a, b), integral(f, b, c), integral(f, a, c))
        integral(f, a, b) + integral(f, b, c) <= integral(f, a, c)
        inf_upper_add_ge(f, a, b, c, integral(f, a, b), integral(f, b, c), integral(f, a, c))
        integral(f, a, c) <= integral(f, a, b) + integral(f, b, c)
        lte_antisymm[Real](integral(f, a, c), integral(f, a, b) + integral(f, b, c))
        integral(f, a, c) = integral(f, a, b) + integral(f, b, c)
    }
}

/// Additivity of the integral over adjacent intervals, with the sum on the
/// left-hand side.
theorem integral_additivity_symm(f: Real -> Real, a: Real, b: Real, c: Real) {
    a <= b and b <= c and is_integrable(f, a, b) and is_integrable(f, b, c) and is_integrable(f, a, c)
    implies integral(f, a, b) + integral(f, b, c) = integral(f, a, c)
} by {
    if a <= b and b <= c and is_integrable(f, a, b) and is_integrable(f, b, c) and is_integrable(f, a, c) {
        integral_additivity(f, a, b, c)
        integral(f, a, c) = integral(f, a, b) + integral(f, b, c)
        integral(f, a, b) + integral(f, b, c) = integral(f, a, c)
    }
}
