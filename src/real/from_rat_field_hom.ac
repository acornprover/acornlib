from rat import Rat
from real.real_field import Real
from real.real_base import add_from_rat, from_rat_maintains_lt, from_rat_maintains_lte
from real.real_ring import mul_from_rat, real_lte_imp_rat_lte
from algebra.ring.ring_hom import RingHom, is_ring_hom
from algebra.field.field_hom import FieldHom, is_field_hom, field_hom_injective,
    field_hom_inverse

/// The rational-to-real embedding as a named function.
define real_from_rat_fn(r: Rat) -> Real {
    Real.from_rat(r)
}

/// The rational-to-real embedding preserves addition pointwise.
theorem real_from_rat_fn_add(p: Rat, q: Rat) {
    real_from_rat_fn(p + q) = real_from_rat_fn(p) + real_from_rat_fn(q)
} by {
    real_from_rat_fn(p + q) = Real.from_rat(p + q)
    real_from_rat_fn(p) = Real.from_rat(p)
    real_from_rat_fn(q) = Real.from_rat(q)
    add_from_rat(p, q)
    Real.from_rat(p) + Real.from_rat(q) = Real.from_rat(p + q)
    real_from_rat_fn(p) + real_from_rat_fn(q) = real_from_rat_fn(p + q)
}

/// The rational-to-real embedding preserves multiplication pointwise.
theorem real_from_rat_fn_mul(p: Rat, q: Rat) {
    real_from_rat_fn(p * q) = real_from_rat_fn(p) * real_from_rat_fn(q)
} by {
    real_from_rat_fn(p * q) = Real.from_rat(p * q)
    real_from_rat_fn(p) = Real.from_rat(p)
    real_from_rat_fn(q) = Real.from_rat(q)
    mul_from_rat(p, q)
    Real.from_rat(p) * Real.from_rat(q) = Real.from_rat(p * q)
    real_from_rat_fn(p) * real_from_rat_fn(q) = real_from_rat_fn(p * q)
}

/// The rational-to-real embedding sends one to one.
theorem real_from_rat_fn_one {
    real_from_rat_fn(Rat.1) = Real.1
} by {
    real_from_rat_fn(Rat.1) = Real.from_rat(Rat.1)
    Real.1 = Real.from_rat(Rat.1)
}

/// The rational-to-real embedding preserves the ring operations and one.
theorem real_from_rat_is_ring_hom {
    is_ring_hom(real_from_rat_fn)
} by {
    real_from_rat_fn_one
    forall(p: Rat, q: Rat) {
        real_from_rat_fn_add(p, q)
        real_from_rat_fn_mul(p, q)
        real_from_rat_fn(p + q) = real_from_rat_fn(p) + real_from_rat_fn(q) and
            real_from_rat_fn(p * q) = real_from_rat_fn(p) * real_from_rat_fn(q)
    }
    real_from_rat_fn(Rat.1) = Real.1 and forall(p: Rat, q: Rat) {
        real_from_rat_fn(p + q) = real_from_rat_fn(p) + real_from_rat_fn(q) and
            real_from_rat_fn(p * q) = real_from_rat_fn(p) * real_from_rat_fn(q)
    }
    is_ring_hom(real_from_rat_fn)
}

/// `RingHom.new` produces a `Some` value for the rational-to-real embedding.
theorem real_from_rat_ring_hom_some {
    exists(h: RingHom[Rat, Real]) {
        RingHom.new(real_from_rat_fn) = Option.some(h)
    }
} by {
    real_from_rat_is_ring_hom
}

/// The rational-to-real embedding as a bundled ring homomorphism.
let real_from_rat_ring_hom: RingHom[Rat, Real] satisfy {
    RingHom.new(real_from_rat_fn) = Option.some(real_from_rat_ring_hom)
}

/// The underlying function of the bundled rational-to-real ring homomorphism.
theorem real_from_rat_ring_hom_hom {
    real_from_rat_ring_hom.hom = real_from_rat_fn
} by {
    RingHom.new(real_from_rat_fn) = Option.some(real_from_rat_ring_hom)
}

/// Applying the bundled ring homomorphism is the rational-to-real embedding.
theorem real_from_rat_ring_hom_apply(p: Rat) {
    real_from_rat_ring_hom.hom(p) = Real.from_rat(p)
} by {
    real_from_rat_ring_hom_hom
    real_from_rat_ring_hom.hom = real_from_rat_fn
    real_from_rat_fn(p) = Real.from_rat(p)
}

/// The rational-to-real embedding is a field homomorphism.
theorem real_from_rat_is_field_hom {
    is_field_hom(real_from_rat_fn)
} by {
    real_from_rat_is_ring_hom
}

/// `FieldHom.new` produces a `Some` value for the rational-to-real embedding.
theorem real_from_rat_field_hom_some {
    exists(h: FieldHom[Rat, Real]) {
        FieldHom.new(real_from_rat_fn) = Option.some(h)
    }
} by {
    real_from_rat_is_field_hom
}

/// The rational-to-real embedding as a bundled field homomorphism.
let real_from_rat_field_hom: FieldHom[Rat, Real] satisfy {
    FieldHom.new(real_from_rat_fn) = Option.some(real_from_rat_field_hom)
}

/// The underlying function of the bundled rational-to-real field homomorphism.
theorem real_from_rat_field_hom_hom {
    real_from_rat_field_hom.hom = real_from_rat_fn
} by {
    FieldHom.new(real_from_rat_fn) = Option.some(real_from_rat_field_hom)
}

/// Applying the bundled field homomorphism is the rational-to-real embedding.
theorem real_from_rat_field_hom_apply(p: Rat) {
    real_from_rat_field_hom.hom(p) = Real.from_rat(p)
} by {
    real_from_rat_field_hom_hom
    real_from_rat_field_hom.hom = real_from_rat_fn
    real_from_rat_fn(p) = Real.from_rat(p)
}

/// The bundled rational-to-real field homomorphism is injective.
theorem real_from_rat_field_hom_injective(p: Rat, q: Rat) {
    real_from_rat_field_hom.hom(p) = real_from_rat_field_hom.hom(q) implies p = q
} by {
    if real_from_rat_field_hom.hom(p) = real_from_rat_field_hom.hom(q) {
        field_hom_injective(real_from_rat_field_hom, p, q)
    }
}

/// The bundled rational-to-real field homomorphism preserves nonzero inverses.
theorem real_from_rat_field_hom_inverse(p: Rat) {
    p != Rat.0 implies
        real_from_rat_field_hom.hom(p.inverse) = real_from_rat_field_hom.hom(p).inverse
} by {
    if p != Rat.0 {
        field_hom_inverse(real_from_rat_field_hom, p)
    }
}

/// The rational-to-real embedding preserves nonzero inverses.
theorem real_from_rat_inverse(p: Rat) {
    p != Rat.0 implies Real.from_rat(p.inverse) = Real.from_rat(p).inverse
} by {
    if p != Rat.0 {
        real_from_rat_field_hom_inverse(p)
        real_from_rat_field_hom_hom
        real_from_rat_field_hom.hom = real_from_rat_fn
        real_from_rat_fn(p.inverse) = Real.from_rat(p.inverse)
        real_from_rat_fn(p) = Real.from_rat(p)
        real_from_rat_field_hom.hom(p.inverse) = Real.from_rat(p.inverse)
        real_from_rat_field_hom.hom(p) = Real.from_rat(p)
        Real.from_rat(p.inverse) = Real.from_rat(p).inverse
    }
}

/// The rational-to-real embedding is injective.
theorem real_from_rat_injective(p: Rat, q: Rat) {
    Real.from_rat(p) = Real.from_rat(q) implies p = q
} by {
    if Real.from_rat(p) = Real.from_rat(q) {
        real_from_rat_field_hom_hom
        real_from_rat_field_hom.hom = real_from_rat_fn
        real_from_rat_fn(p) = Real.from_rat(p)
        real_from_rat_fn(q) = Real.from_rat(q)
        real_from_rat_field_hom.hom(p) = real_from_rat_fn(p)
        real_from_rat_field_hom.hom(q) = real_from_rat_fn(q)
        real_from_rat_field_hom.hom(p) = real_from_rat_field_hom.hom(q)
        field_hom_injective(real_from_rat_field_hom, p, q)
    }
}

/// Equal rational inputs have equal real embeddings.
theorem real_from_rat_eq_of_eq(p: Rat, q: Rat) {
    p = q implies Real.from_rat(p) = Real.from_rat(q)
} by {
    if p = q {
        Real.from_rat(p) = Real.from_rat(q)
    }
}

/// The rational-to-real embedding preserves strict order.
theorem real_from_rat_preserves_lt(p: Rat, q: Rat) {
    p < q implies Real.from_rat(p) < Real.from_rat(q)
} by {
    if p < q {
        from_rat_maintains_lt(p, q)
    }
}

/// The rational-to-real embedding preserves non-strict order.
theorem real_from_rat_preserves_lte(p: Rat, q: Rat) {
    p <= q implies Real.from_rat(p) <= Real.from_rat(q)
} by {
    if p <= q {
        from_rat_maintains_lte(p, q)
    }
}

/// The rational-to-real embedding reflects strict order.
theorem real_from_rat_reflects_lt(p: Rat, q: Rat) {
    Real.from_rat(p) < Real.from_rat(q) implies p < q
} by {
    if Real.from_rat(p) < Real.from_rat(q) {
        Real.from_rat(p) <= Real.from_rat(q)
        real_lte_imp_rat_lte(p, q)
        p <= q
        if p = q {
            Real.from_rat(p) = Real.from_rat(q)
            false
        }
        p != q
        p < q
    }
}

/// The rational-to-real embedding reflects non-strict order.
theorem real_from_rat_reflects_lte(p: Rat, q: Rat) {
    Real.from_rat(p) <= Real.from_rat(q) implies p <= q
} by {
    if Real.from_rat(p) <= Real.from_rat(q) {
        real_lte_imp_rat_lte(p, q)
    }
}
