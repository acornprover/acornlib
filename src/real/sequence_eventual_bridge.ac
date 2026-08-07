/// Bridges real-sequence eventual membership with generic Nat-eventual predicates.
///
/// This support layer is intentionally separate from public asymptotic notation.

from nat import Nat, lte_trans
from order import max_imp_gte
from data.basic.set import Set
from data.basic.relation_transport import predicate_pullback, predicate_subset
from data.nat.nat_eventually import eventually_after_nat, eventually_after_nat_at,
    eventually_predicate_nat, eventually_predicate_nat_has_witness,
    eventually_predicate_nat_intro, eventually_predicate_nat_monotone,
    eventually_predicate_nat_of_forall
from real.real_field import Real
from real.sequence_set_membership import seq_eventually_in_real_set
from real.sequence_tail_sets import sequence_tail_set, seq_eventually_in_real_set_iff_tail_subset

/// Pointwise equality predicate for two real sequences.
define real_sequence_eq_predicate(a: Nat -> Real, b: Nat -> Real, n: Nat) -> Bool {
    a(n) = b(n)
}

/// True if two real sequences are equal for all sufficiently late indices.
define seq_eventually_equal_real(a: Nat -> Real, b: Nat -> Real) -> Bool {
    eventually_predicate_nat(real_sequence_eq_predicate(a, b))
}

/// Eventual membership in a real set is generic Nat-eventuality of the pulled-back set predicate.
theorem seq_eventually_in_real_set_iff_pullback_eventually(s: Set[Real], a: Nat -> Real) {
    seq_eventually_in_real_set(s, a) = eventually_predicate_nat(predicate_pullback(a, s.contains))
} by {
    if seq_eventually_in_real_set(s, a) {
        let n0: Nat satisfy {
            forall(n: Nat) {
                n0 <= n implies s.contains(a(n))
            }
        }
        forall(n: Nat) {
            if n0 <= n {
                s.contains(a(n))
                predicate_pullback(a, s.contains, n)
            }
        }
        eventually_after_nat(predicate_pullback(a, s.contains), n0)
        eventually_predicate_nat_intro(predicate_pullback(a, s.contains), n0)
        eventually_predicate_nat(predicate_pullback(a, s.contains))
    }
    if eventually_predicate_nat(predicate_pullback(a, s.contains)) {
        eventually_predicate_nat_has_witness(predicate_pullback(a, s.contains))
        let n0: Nat satisfy {
            eventually_after_nat(predicate_pullback(a, s.contains), n0)
        }
        forall(n: Nat) {
            if n0 <= n {
                eventually_after_nat_at(predicate_pullback(a, s.contains), n0, n)
                predicate_pullback(a, s.contains, n)
                s.contains(a(n))
            }
        }
        exists(m: Nat) {
            forall(n: Nat) {
                m <= n implies s.contains(a(n))
            }
        }
        seq_eventually_in_real_set(s, a)
    }
    seq_eventually_in_real_set(s, a) = eventually_predicate_nat(predicate_pullback(a, s.contains))
}

/// Generic Nat-eventuality of sequence-set membership is equivalent to containing a tail range.
theorem eventually_pullback_iff_sequence_tail_subset(s: Set[Real], a: Nat -> Real) {
    eventually_predicate_nat(predicate_pullback(a, s.contains)) = exists(m: Nat) {
        sequence_tail_set(a, m).subset(s)
    }
} by {
    if eventually_predicate_nat(predicate_pullback(a, s.contains)) {
        seq_eventually_in_real_set_iff_pullback_eventually(s, a)
        seq_eventually_in_real_set(s, a)
        seq_eventually_in_real_set_iff_tail_subset(s, a)
        exists(m: Nat) {
            sequence_tail_set(a, m).subset(s)
        }
    }
    if exists(m: Nat) { sequence_tail_set(a, m).subset(s) } {
        seq_eventually_in_real_set_iff_tail_subset(s, a)
        seq_eventually_in_real_set(s, a)
        seq_eventually_in_real_set_iff_pullback_eventually(s, a)
        eventually_predicate_nat(predicate_pullback(a, s.contains))
    }
    eventually_predicate_nat(predicate_pullback(a, s.contains)) = exists(m: Nat) {
        sequence_tail_set(a, m).subset(s)
    }
}

/// Eventual real-sequence equality is reflexive.
theorem seq_eventually_equal_real_refl(a: Nat -> Real) {
    seq_eventually_equal_real(a, a)
} by {
    forall(n: Nat) {
        real_sequence_eq_predicate(a, a, n)
    }
    eventually_predicate_nat_of_forall(real_sequence_eq_predicate(a, a))
    eventually_predicate_nat(real_sequence_eq_predicate(a, a))
    seq_eventually_equal_real(a, a)
}

/// Eventual real-sequence equality is symmetric.
theorem seq_eventually_equal_real_symm(a: Nat -> Real, b: Nat -> Real) {
    seq_eventually_equal_real(a, b) implies seq_eventually_equal_real(b, a)
} by {
    if seq_eventually_equal_real(a, b) {
        eventually_predicate_nat_has_witness(real_sequence_eq_predicate(a, b))
        let n0: Nat satisfy {
            eventually_after_nat(real_sequence_eq_predicate(a, b), n0)
        }
        forall(n: Nat) {
            if n0 <= n {
                eventually_after_nat_at(real_sequence_eq_predicate(a, b), n0, n)
                real_sequence_eq_predicate(a, b, n)
                a(n) = b(n)
                b(n) = a(n)
                real_sequence_eq_predicate(b, a, n)
            }
        }
        eventually_after_nat(real_sequence_eq_predicate(b, a), n0)
        eventually_predicate_nat_intro(real_sequence_eq_predicate(b, a), n0)
        eventually_predicate_nat(real_sequence_eq_predicate(b, a))
        seq_eventually_equal_real(b, a)
    }
}

/// Eventual real-sequence equality transports eventual membership in real sets.
theorem seq_eventually_equal_real_preserves_eventual_membership(
    s: Set[Real],
    a: Nat -> Real,
    b: Nat -> Real
) {
    seq_eventually_equal_real(a, b) and seq_eventually_in_real_set(s, a)
    implies seq_eventually_in_real_set(s, b)
} by {
    if seq_eventually_equal_real(a, b) and seq_eventually_in_real_set(s, a) {
        eventually_predicate_nat_has_witness(real_sequence_eq_predicate(a, b))
        let ne: Nat satisfy {
            eventually_after_nat(real_sequence_eq_predicate(a, b), ne)
        }
        let ns: Nat satisfy {
            forall(n: Nat) {
                ns <= n implies s.contains(a(n))
            }
        }
        let n0 = ne.max(ns)
        max_imp_gte[Nat](ne, ns)
        ne.max(ns) >= ne and ne.max(ns) >= ns
        ne <= n0
        ns <= n0
        forall(n: Nat) {
            if n0 <= n {
                lte_trans(ne, n0, n)
                ne <= n
                lte_trans(ns, n0, n)
                ns <= n
                eventually_after_nat_at(real_sequence_eq_predicate(a, b), ne, n)
                real_sequence_eq_predicate(a, b, n)
                a(n) = b(n)
                s.contains(a(n))
                s.contains(b(n))
            }
        }
        exists(m: Nat) {
            forall(n: Nat) {
                m <= n implies s.contains(b(n))
            }
        }
        seq_eventually_in_real_set(s, b)
    }
}

/// Sequence-eventual membership is preserved by predicate implication along the sequence.
theorem seq_eventually_in_real_set_of_pullback_subset(s: Set[Real], t: Set[Real], a: Nat -> Real) {
    predicate_subset(predicate_pullback(a, s.contains), predicate_pullback(a, t.contains)) and
    seq_eventually_in_real_set(s, a) implies seq_eventually_in_real_set(t, a)
} by {
    if predicate_subset(predicate_pullback(a, s.contains), predicate_pullback(a, t.contains)) and
    seq_eventually_in_real_set(s, a) {
        seq_eventually_in_real_set_iff_pullback_eventually(s, a)
        eventually_predicate_nat(predicate_pullback(a, s.contains))
        eventually_predicate_nat_monotone(predicate_pullback(a, s.contains), predicate_pullback(a, t.contains))
        eventually_predicate_nat(predicate_pullback(a, t.contains))
        seq_eventually_in_real_set_iff_pullback_eventually(t, a)
        seq_eventually_in_real_set(t, a)
    }
}
