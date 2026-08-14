/// Fixed point theorems for continuous real functions on closed intervals.
///
/// A continuous self-map of a closed interval has a fixed point: this is the
/// intermediate value theorem applied to `x ↦ x - f(x)`.  The one-dimensional
/// case of Brouwer's fixed point theorem, for the unit interval, is the
/// special case with endpoints zero and one.

from order import lte_refl, lte_trans, lte_antisymm, lt_imp_lte
from order_set import closed_interval_set, closed_interval_set_contains_eq,
    closed_interval_set_contains_lower, closed_interval_set_contains_upper,
    closed_interval_set_lower_le, closed_interval_set_le_upper
from real.continuity_base import Real, continuous
from real.continuity_sub_fns import sub_fns, sub_fns_continuous
from real.continuity_sequences import identity_function_is_continuous
from real.intermediate_value import intermediate_value_closed_interval
from real.real_seq import sub_zero_imp_eq
from real.integral import sub_nonneg
from algebra.add_ordered_group import add_le_add_right
from data.basic.functions import identity_fn

numerals Real

/// A continuous self-map of a closed interval has a fixed point in the interval.
///
/// This is the classical corollary of the intermediate value theorem: the map
/// `x ↦ x - f(x)` is nonpositive at `lower` and nonnegative at `upper`, so it
/// vanishes somewhere in the interval.
theorem interval_fixed_point(f: Real -> Real, lower: Real, upper: Real) {
    continuous(f)
    and lower <= upper
    and (forall(x: Real) {
        closed_interval_set(lower, upper).contains(x) implies
            closed_interval_set(lower, upper).contains(f(x))
    })
    implies exists(point: Real) {
        closed_interval_set(lower, upper).contains(point) and f(point) = point
    }
} by {
    if continuous(f) and lower <= upper and (forall(x: Real) {
        closed_interval_set(lower, upper).contains(x) implies
            closed_interval_set(lower, upper).contains(f(x))
    }) {
        identity_function_is_continuous
        continuous(identity_fn[Real])
        sub_fns_continuous(identity_fn[Real], f)
        continuous(sub_fns(identity_fn[Real], f))
        closed_interval_set_contains_lower[Real](lower, upper)
        closed_interval_set(lower, upper).contains(lower)
        closed_interval_set(lower, upper).contains(f(lower))
        closed_interval_set_lower_le[Real](lower, upper, f(lower))
        lower <= f(lower)
        add_le_add_right[Real](lower, f(lower), -f(lower))
        lower + -f(lower) <= f(lower) + -f(lower)
        f(lower) + -f(lower) = Real.0
        lower + -f(lower) <= Real.0
        sub_fns(identity_fn[Real], f, lower) = lower - f(lower)
        lower - f(lower) = lower + -f(lower)
        sub_fns(identity_fn[Real], f, lower) = lower + -f(lower)
        sub_fns(identity_fn[Real], f, lower) <= Real.0
        closed_interval_set_contains_upper[Real](lower, upper)
        closed_interval_set(lower, upper).contains(upper)
        closed_interval_set(lower, upper).contains(f(upper))
        closed_interval_set_le_upper[Real](lower, upper, f(upper))
        f(upper) <= upper
        sub_nonneg(f(upper), upper)
        Real.0 <= upper - f(upper)
        sub_fns(identity_fn[Real], f, upper) = upper - f(upper)
        Real.0 <= sub_fns(identity_fn[Real], f, upper)
        intermediate_value_closed_interval(sub_fns(identity_fn[Real], f), lower, upper, Real.0)
        exists(root: Real) {
            closed_interval_set(lower, upper).contains(root)
            and sub_fns(identity_fn[Real], f, root) = Real.0
        }
        let root: Real satisfy {
            closed_interval_set(lower, upper).contains(root)
            and sub_fns(identity_fn[Real], f, root) = Real.0
        }
        closed_interval_set(lower, upper).contains(root)
        sub_fns(identity_fn[Real], f, root) = Real.0
        sub_fns(identity_fn[Real], f, root) = root - f(root)
        root - f(root) = Real.0
        sub_zero_imp_eq(root, f(root))
        root = f(root)
        closed_interval_set(lower, upper).contains(root) and f(root) = root
        exists(fixed: Real) {
            closed_interval_set(lower, upper).contains(fixed) and f(fixed) = fixed
        }
    }
}

/// A continuous self-map of the unit interval has a fixed point.
///
/// This is the one-dimensional case of Brouwer's fixed point theorem: every
/// continuous map from the closed unit interval to itself has a fixed point.
theorem unit_interval_fixed_point(f: Real -> Real) {
    continuous(f)
    and (forall(x: Real) {
        closed_interval_set(Real.0, Real.1).contains(x) implies
            closed_interval_set(Real.0, Real.1).contains(f(x))
    })
    implies exists(point: Real) {
        closed_interval_set(Real.0, Real.1).contains(point) and f(point) = point
    }
} by {
    if continuous(f) and (forall(x: Real) {
        closed_interval_set(Real.0, Real.1).contains(x) implies
            closed_interval_set(Real.0, Real.1).contains(f(x))
    }) {
        Real.1 > Real.0
        lt_imp_lte(Real.0, Real.1)
        Real.0 <= Real.1
        interval_fixed_point(f, Real.0, Real.1)
        exists(point: Real) {
            closed_interval_set(Real.0, Real.1).contains(point) and f(point) = point
        }
    }
}
