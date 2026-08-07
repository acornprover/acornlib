from data.basic.set import Set, difference_subset, sets_subset_intersection
from real.real_field import Real
from real.topology import is_bounded_real_set, subset_of_bounded_real_set_is_bounded

/// Intersecting a bounded real set on the right preserves boundedness.
theorem intersection_left_of_bounded_real_set_is_bounded(s: Set[Real], t: Set[Real]) {
    is_bounded_real_set(s) implies is_bounded_real_set(s.intersection(t))
} by {
    if is_bounded_real_set(s) {
        sets_subset_intersection[Real](s, t)
        s.intersection(t).subset(s)
        subset_of_bounded_real_set_is_bounded(s.intersection(t), s)
        is_bounded_real_set(s.intersection(t))
    }
}

/// Intersecting a bounded real set on the left preserves boundedness.
theorem intersection_right_of_bounded_real_set_is_bounded(s: Set[Real], t: Set[Real]) {
    is_bounded_real_set(t) implies is_bounded_real_set(s.intersection(t))
} by {
    if is_bounded_real_set(t) {
        sets_subset_intersection[Real](s, t)
        s.intersection(t).subset(t)
        subset_of_bounded_real_set_is_bounded(s.intersection(t), t)
        is_bounded_real_set(s.intersection(t))
    }
}

/// The intersection of two bounded real sets is bounded.
theorem intersection_of_bounded_real_sets_is_bounded(s: Set[Real], t: Set[Real]) {
    is_bounded_real_set(s) and is_bounded_real_set(t) implies is_bounded_real_set(s.intersection(t))
} by {
    if is_bounded_real_set(s) and is_bounded_real_set(t) {
        intersection_left_of_bounded_real_set_is_bounded(s, t)
        is_bounded_real_set(s.intersection(t))
    }
}

/// Removing an arbitrary set from a bounded real set preserves boundedness.
theorem difference_of_bounded_real_set_is_bounded(s: Set[Real], t: Set[Real]) {
    is_bounded_real_set(s) implies is_bounded_real_set(s.difference(t))
} by {
    if is_bounded_real_set(s) {
        difference_subset[Real](s, t)
        s.difference(t).subset(s)
        subset_of_bounded_real_set_is_bounded(s.difference(t), s)
        is_bounded_real_set(s.difference(t))
    }
}
