from nat import Nat, alt_induction, zero_or_suc, lt_and_lte, lt_suc, lt_suc_right, only_zero_lte_zero, not_lt_zero, from_nat, from_nat_add, from_nat_zero, from_nat_one
from list import partial, partial_split_last, partial_zero, partial_add, partial_scalar_mul, partial_pointwise_eq
from order import lte_antisymm, lte_trans, lt_of_lt_of_lte, lt_of_lte_of_lt, lt_trans, not_lte_imp_gt, not_gte_imp_lt, not_lt_imp_gte, not_gt_imp_lte, lt_irrefl
from data.basic.logic import not_forall_imp_exists_not, not_implies
from algebra.add_semigroup import add_fn
from algebra.semigroup import mul_fn
from algebra.add_comm_group import sub_eq_zero_imp_eq
from real.cauchy_schwarz import real_square_add_expanded
from real.real_base import one_half_positive, one_half_plus_one_half, lte_add_right, lte_add_left, lt_add_right, lt_add_left, add_lte_add, add_neg_eq_zero, pos_gt_zero, gt_zero_imp_pos, neg_neg
from real.real_ring import mul_nonneg, mul_pos_pos, lte_mul_nonneg_left, lte_mul_nonneg_right, mul_le_mul_nonneg, real_mul_comm, mul_assoc, mul_distrib_right, mul_distrib_left, mul_one_left, mul_one_right, mul_zero_left, mul_zero_right, mul_neg_left, mul_neg_right, mul_neg_neg, mul_neg_one_left, square_nonneg, lt_mul_pos_right
from real.real_field import mul_div_cancel, mul_inverse, div_le_of_mul_le, mul_le_mul_pos_right, mul_le_mul_pos_left, mul_div
from real.harmonic import real_one_div_pos, real_recip_antitone_pos, real_inverse_antitone_pos_strict, from_nat_suc_pos_real
from real.exp import Real, exp_gt_one_plus_x, exp_add, exp_pos, exp_zero, pow_suc, pow_pos, zero_pow_pos, mul_frac_assoc, mul_one_over, mul_assoc_real
from real.exp_inequalities import exp_monotone, one_div_one_div, exp_ge_one_plus_self
from real.log import log_mul, log_some_of_pos_exists, log_exp, log_one, exp_log_or_zero, exp_neg, exp_injective, exp_nat_mul, rpow_zero_base_pos, rpow_nat, exp_le_geometric_recip
from real.log_inequalities import log_le_sub_one, log_ne_zero_of_pos_ne_one
from real.finite_product_mean import finite_real_product, finite_real_mean, nonnegative_on, finite_real_product_suc, finite_real_product_zero, nonnegative_on_prefix, finite_real_product_nonnegative, from_nat_real_pos_of_ne_zero, from_nat_real_ne_zero_of_ne_zero, finite_real_mean_mul_count

numerals Nat
numerals Real

/// The first `n` values of a real sequence are positive.
define positive_on(f: Nat -> Real, n: Nat) -> Bool {
    forall(i: Nat) { i < n implies f(i) > Real.0 }
}

/// Restricted positivity is inherited by shorter initial segments.
theorem positive_on_prefix(f: Nat -> Real, n: Nat, m: Nat) {
    positive_on(f, n) and m <= n implies positive_on(f, m)
} by {
    if positive_on(f, n) and m <= n {
        forall(i: Nat) {
            if i < m {
                lt_and_lte(i, m, n)
                i < n
                positive_on(f, n) = forall(j: Nat) { j < n implies f(j) > Real.0 }
                f(i) > Real.0
            }
        }
    }
}

/// Positivity on a prefix implies nonnegativity on it.
theorem positive_on_imp_nonnegative_on(f: Nat -> Real, n: Nat) {
    positive_on(f, n) implies nonnegative_on(f, n)
} by {
    if positive_on(f, n) {
        positive_on(f, n) = forall(i: Nat) { i < n implies f(i) > Real.0 }
        forall(i: Nat) {
            if i < n {
                f(i) > Real.0
                f(i) >= Real.0
            }
        }
    }
}

/// A finite product over a positive prefix is positive.
theorem finite_real_product_pos(f: Nat -> Real, n: Nat) {
    positive_on(f, n) implies finite_real_product(f, n) > Real.0
} by {
    define p(k: Nat) -> Bool {
        positive_on(f, k) implies finite_real_product(f, k) > Real.0
    }
    finite_real_product_zero(f)
    Real.1 > Real.0
    p(Nat.0)
    forall(k: Nat) {
        if p(k) {
            if positive_on(f, k.suc) {
                k < k.suc
                k <= k.suc
                positive_on_prefix(f, k.suc, k)
                positive_on(f, k)
                p(k) = (positive_on(f, k) implies finite_real_product(f, k) > Real.0)
                finite_real_product(f, k) > Real.0
                positive_on(f, k.suc) = forall(i: Nat) { i < k.suc implies f(i) > Real.0 }
                f(k) > Real.0
                gt_zero_imp_pos(finite_real_product(f, k))
                gt_zero_imp_pos(f(k))
                mul_pos_pos(finite_real_product(f, k), f(k))
                (finite_real_product(f, k) * f(k)).is_positive
                pos_gt_zero(finite_real_product(f, k) * f(k))
                finite_real_product(f, k) * f(k) > Real.0
                finite_real_product_suc(f, k)
                finite_real_product(f, k.suc) = finite_real_product(f, k) * f(k)
                finite_real_product(f, k.suc) > Real.0
            }
            p(k.suc)
        }
    }
    p(Nat.0) and forall(k: Nat) { p(k) implies p(k.suc) }
    alt_induction(p)
    forall(k: Nat) { p(k) }
    p(n)
}

/// A partial sum over a nonnegative prefix is nonnegative.
theorem partial_nonneg_bounded(f: Nat -> Real, n: Nat) {
    nonnegative_on(f, n) implies partial(f, n) >= Real.0
} by {
    define p(k: Nat) -> Bool {
        nonnegative_on(f, k) implies partial(f, k) >= Real.0
    }
    partial_zero(f)
    partial(f, Nat.0) = Real.0
    Real.0 >= Real.0
    p(Nat.0)
    forall(k: Nat) {
        if p(k) {
            if nonnegative_on(f, k.suc) {
                k < k.suc
                k <= k.suc
                nonnegative_on_prefix(f, k.suc, k)
                nonnegative_on(f, k)
                p(k) = (nonnegative_on(f, k) implies partial(f, k) >= Real.0)
                partial(f, k) >= Real.0
                nonnegative_on(f, k.suc) = forall(i: Nat) { i < k.suc implies f(i) >= Real.0 }
                f(k) >= Real.0
                lte_add_right(Real.0, partial(f, k), f(k))
                Real.0 + f(k) <= partial(f, k) + f(k)
                Real.0 + f(k) = f(k)
                f(k) <= partial(f, k) + f(k)
                lte_trans(Real.0, f(k), partial(f, k) + f(k))
                Real.0 <= partial(f, k) + f(k)
                partial_split_last(f, k)
                partial(f, k.suc) = partial(f, k) + f(k)
                partial(f, k.suc) >= Real.0
            }
            p(k.suc)
        }
    }
    p(Nat.0) and forall(k: Nat) { p(k) implies p(k.suc) }
    alt_induction(p)
    forall(k: Nat) { p(k) }
    p(n)
}

/// Adding a positive real to a nonnegative one is positive.
theorem add_nonneg_pos(a: Real, b: Real) {
    a >= Real.0 and b > Real.0 implies a + b > Real.0
} by {
    if a >= Real.0 and b > Real.0 {
        lte_add_right(Real.0, a, b)
        Real.0 + b <= a + b
        Real.0 + b = b
        b <= a + b
        Real.0 < b
        lt_of_lt_of_lte(Real.0, b, a + b)
        Real.0 < a + b
        a + b > Real.0
    }
}

/// A partial sum over a positive prefix with a nonzero count is positive.
theorem partial_pos_bounded(f: Nat -> Real, n: Nat) {
    positive_on(f, n) and n != Nat.0 implies partial(f, n) > Real.0
} by {
    if positive_on(f, n) and n != Nat.0 {
        zero_or_suc(n)
        let k: Nat satisfy {
            n = k.suc
        }
        k < k.suc
        k <= k.suc
        positive_on_prefix(f, n, k)
        positive_on(f, k)
        positive_on_imp_nonnegative_on(f, k)
        nonnegative_on(f, k)
        partial_nonneg_bounded(f, k)
        partial(f, k) >= Real.0
        positive_on(f, n) = forall(i: Nat) { i < n implies f(i) > Real.0 }
        f(k) > Real.0
        add_nonneg_pos(partial(f, k), f(k))
        partial(f, k) + f(k) > Real.0
        partial_split_last(f, k)
        partial(f, k.suc) = partial(f, k) + f(k)
        partial(f, n) > Real.0
    }
}

/// The reciprocal of a positive real is positive.
theorem recip_pos(x: Real) {
    x > Real.0 implies x.inverse > Real.0
} by {
    if x > Real.0 {
        real_one_div_pos(x)
        Real.1 / x > Real.0
        Real.1 / x = x.inverse
        x.inverse > Real.0
    }
}

/// A quotient of a positive real by a positive real is positive.
theorem div_pos_of_pos_pos(x: Real, c: Real) {
    x > Real.0 and c > Real.0 implies x / c > Real.0
} by {
    if x > Real.0 and c > Real.0 {
        recip_pos(c)
        c.inverse > Real.0
        gt_zero_imp_pos(x)
        gt_zero_imp_pos(c.inverse)
        mul_pos_pos(x, c.inverse)
        (x * c.inverse).is_positive
        pos_gt_zero(x * c.inverse)
        x * c.inverse > Real.0
        x / c = x * c.inverse
        x / c > Real.0
    }
}

/// A quotient of a nonnegative real by a positive real is nonnegative.
theorem div_nonneg_of_nonneg_pos(x: Real, c: Real) {
    x >= Real.0 and c > Real.0 implies x / c >= Real.0
} by {
    if x >= Real.0 and c > Real.0 {
        recip_pos(c)
        c.inverse > Real.0
        c.inverse >= Real.0
        mul_nonneg(x, c.inverse)
        x * c.inverse >= Real.0
        x / c = x * c.inverse
        x / c >= Real.0
    }
}

/// Pointwise logarithms of a real sequence.
define log_value_fn(f: Nat -> Real) -> (Nat -> Real) {
    function(i: Nat) { (f(i)).log.get_or_else(Real.0) }
}

/// The logarithm of a finite product of positive values is the partial sum of
/// their logarithms.
theorem log_finite_product(f: Nat -> Real, n: Nat) {
    positive_on(f, n) implies (finite_real_product(f, n)).log =
        Option.some(partial(log_value_fn(f), n))
} by {
    define p(k: Nat) -> Bool {
        positive_on(f, k) implies (finite_real_product(f, k)).log =
            Option.some(partial(log_value_fn(f), k))
    }
    finite_real_product_zero(f)
    finite_real_product(f, Nat.0) = Real.1
    log_one
    Real.1.log = Option.some(Real.0)
    partial_zero(log_value_fn(f))
    partial(log_value_fn(f), Nat.0) = Real.0
    p(Nat.0)
    forall(k: Nat) {
        if p(k) {
            if positive_on(f, k.suc) {
                k < k.suc
                k <= k.suc
                positive_on_prefix(f, k.suc, k)
                positive_on(f, k)
                p(k) = (positive_on(f, k) implies (finite_real_product(f, k)).log =
                    Option.some(partial(log_value_fn(f), k)))
                (finite_real_product(f, k)).log = Option.some(partial(log_value_fn(f), k))
                positive_on(f, k.suc) = forall(i: Nat) { i < k.suc implies f(i) > Real.0 }
                f(k) > Real.0
                finite_real_product_pos(f, k)
                finite_real_product(f, k) > Real.0
                log_some_of_pos_exists(finite_real_product(f, k))
                let lpk: Real satisfy {
                    (finite_real_product(f, k)).log = Option.some(lpk)
                }
                log_some_of_pos_exists(f(k))
                let lfk: Real satisfy {
                    f(k).log = Option.some(lfk)
                }
                log_mul(finite_real_product(f, k), f(k), lpk, lfk)
                (finite_real_product(f, k) * f(k)).log = Option.some(lpk + lfk)
                option_get_or_else_some[Real](lpk, Real.0)
                option_get_or_else(Option.some(lpk), Real.0) = lpk
                (finite_real_product(f, k)).log.get_or_else(Real.0) = lpk
                option_get_or_else_some[Real](lfk, Real.0)
                option_get_or_else(Option.some(lfk), Real.0) = lfk
                (f(k)).log.get_or_else(Real.0) = lfk
                lpk + lfk = (finite_real_product(f, k)).log.get_or_else(Real.0) + (f(k)).log.get_or_else(Real.0)
                (finite_real_product(f, k) * f(k)).log =
                    Option.some((finite_real_product(f, k)).log.get_or_else(Real.0) + (f(k)).log.get_or_else(Real.0))
                finite_real_product_suc(f, k)
                finite_real_product(f, k.suc) = finite_real_product(f, k) * f(k)
                (finite_real_product(f, k.suc)).log =
                    Option.some((finite_real_product(f, k)).log.get_or_else(Real.0) + (f(k)).log.get_or_else(Real.0))
                option_get_or_else_some[Real](partial(log_value_fn(f), k), Real.0)
                option_get_or_else(Option.some(partial(log_value_fn(f), k)), Real.0) = partial(log_value_fn(f), k)
                (finite_real_product(f, k)).log.get_or_else(Real.0) = partial(log_value_fn(f), k)
                partial_split_last(log_value_fn(f), k)
                partial(log_value_fn(f), k.suc) =
                    partial(log_value_fn(f), k) + log_value_fn(f)(k)
                log_value_fn(f)(k) = (f(k)).log.get_or_else(Real.0)
                partial(log_value_fn(f), k.suc) =
                    partial(log_value_fn(f), k) + (f(k)).log.get_or_else(Real.0)
                (finite_real_product(f, k.suc)).log = Option.some(partial(log_value_fn(f), k.suc))
            }
            p(k.suc)
        }
    }
    p(Nat.0) and forall(k: Nat) { p(k) implies p(k.suc) }
    alt_induction(p)
    forall(k: Nat) { p(k) }
    p(n)
}

/// The logarithm of the reciprocal of a positive real is the negated logarithm.
theorem log_recip(x: Real) {
    x > Real.0 implies (Real.1 / x).log = Option.some(-x.log.get_or_else(Real.0))
} by {
    if x > Real.0 {
        x != Real.0
        mul_inverse(x)
        x * x.inverse = Real.1
        Real.1 / x = x.inverse
        x * (Real.1 / x) = Real.1
        real_one_div_pos(x)
        Real.1 / x > Real.0
        log_some_of_pos_exists(x)
        let lx: Real satisfy {
            x.log = Option.some(lx)
        }
        log_some_of_pos_exists(Real.1 / x)
        let lrx: Real satisfy {
            (Real.1 / x).log = Option.some(lrx)
        }
        log_mul(x, Real.1 / x, lx, lrx)
        (x * (Real.1 / x)).log = Option.some(lx + lrx)
        option_get_or_else_some[Real](lx, Real.0)
        option_get_or_else(Option.some(lx), Real.0) = lx
        x.log.get_or_else(Real.0) = lx
        option_get_or_else_some[Real](lrx, Real.0)
        option_get_or_else(Option.some(lrx), Real.0) = lrx
        (Real.1 / x).log.get_or_else(Real.0) = lrx
        lx + lrx = x.log.get_or_else(Real.0) + (Real.1 / x).log.get_or_else(Real.0)
        (x * (Real.1 / x)).log = Option.some(x.log.get_or_else(Real.0) + (Real.1 / x).log.get_or_else(Real.0))
        Real.1.log = Option.some(x.log.get_or_else(Real.0) + (Real.1 / x).log.get_or_else(Real.0))
        log_one
        Option.some(Real.0) = Option.some(x.log.get_or_else(Real.0) + (Real.1 / x).log.get_or_else(Real.0))
        some_injective[Real](Real.0, x.log.get_or_else(Real.0) + (Real.1 / x).log.get_or_else(Real.0))
        Real.0 = x.log.get_or_else(Real.0) + (Real.1 / x).log.get_or_else(Real.0)
        x.log.get_or_else(Real.0) + (Real.1 / x).log.get_or_else(Real.0) = Real.0
        (x.log.get_or_else(Real.0) + (Real.1 / x).log.get_or_else(Real.0)) + -x.log.get_or_else(Real.0) = Real.0 + -x.log.get_or_else(Real.0)
        (Real.1 / x).log.get_or_else(Real.0) + (x.log.get_or_else(Real.0) + -x.log.get_or_else(Real.0)) = -x.log.get_or_else(Real.0)
        add_neg_eq_zero(x.log.get_or_else(Real.0))
        x.log.get_or_else(Real.0) + -x.log.get_or_else(Real.0) = Real.0
        (Real.1 / x).log.get_or_else(Real.0) + Real.0 = -x.log.get_or_else(Real.0)
        (Real.1 / x).log.get_or_else(Real.0) = -x.log.get_or_else(Real.0)
        option_get_or_else_some[Real](lrx, Real.0)
        option_get_or_else(Option.some(lrx), Real.0) = lrx
        (Real.1 / x).log.get_or_else(Real.0) = lrx
        (Real.1 / x).log = Option.some((Real.1 / x).log.get_or_else(Real.0))
        (Real.1 / x).log = Option.some(-x.log.get_or_else(Real.0))
    }
}

/// The logarithm of a positive quotient is the difference of the logarithms.
theorem log_div(x: Real, y: Real) {
    x > Real.0 and y > Real.0 implies (x / y).log = Option.some(x.log.get_or_else(Real.0) - y.log.get_or_else(Real.0))
} by {
    if x > Real.0 and y > Real.0 {
        y != Real.0
        mul_one_over(x, y)
        x * (Real.1 / y) = x / y
        real_one_div_pos(y)
        Real.1 / y > Real.0
        log_some_of_pos_exists(x)
        let lx: Real satisfy {
            x.log = Option.some(lx)
        }
        log_some_of_pos_exists(Real.1 / y)
        let lry: Real satisfy {
            (Real.1 / y).log = Option.some(lry)
        }
        log_mul(x, Real.1 / y, lx, lry)
        (x * (Real.1 / y)).log = Option.some(lx + lry)
        option_get_or_else_some[Real](lx, Real.0)
        option_get_or_else(Option.some(lx), Real.0) = lx
        x.log.get_or_else(Real.0) = lx
        option_get_or_else_some[Real](lry, Real.0)
        option_get_or_else(Option.some(lry), Real.0) = lry
        (Real.1 / y).log.get_or_else(Real.0) = lry
        lx + lry = x.log.get_or_else(Real.0) + (Real.1 / y).log.get_or_else(Real.0)
        (x * (Real.1 / y)).log = Option.some(x.log.get_or_else(Real.0) + (Real.1 / y).log.get_or_else(Real.0))
        log_recip(y)
        (Real.1 / y).log = Option.some(-y.log.get_or_else(Real.0))
        option_get_or_else_some[Real](lry, Real.0)
        option_get_or_else(Option.some(lry), Real.0) = lry
        (Real.1 / y).log.get_or_else(Real.0) = lry
        (Real.1 / y).log = Option.some((Real.1 / y).log.get_or_else(Real.0))
        Option.some((Real.1 / y).log.get_or_else(Real.0)) = Option.some(-y.log.get_or_else(Real.0))
        some_injective[Real]((Real.1 / y).log.get_or_else(Real.0), -y.log.get_or_else(Real.0))
        (Real.1 / y).log.get_or_else(Real.0) = -y.log.get_or_else(Real.0)
        (x / y).log = Option.some(x.log.get_or_else(Real.0) + -y.log.get_or_else(Real.0))
        x.log.get_or_else(Real.0) + -y.log.get_or_else(Real.0) = x.log.get_or_else(Real.0) - y.log.get_or_else(Real.0)
        (x / y).log = Option.some(x.log.get_or_else(Real.0) - y.log.get_or_else(Real.0))
    }
}

/// Pointwise comparison on a bounded range compares partial sums.
theorem partial_le_range(f: Nat -> Real, g: Nat -> Real, n: Nat) {
    (forall(i: Nat) { i < n implies f(i) <= g(i) }) implies partial(f, n) <= partial(g, n)
} by {
    define p(k: Nat) -> Bool {
        (forall(i: Nat) { i < k implies f(i) <= g(i) }) implies partial(f, k) <= partial(g, k)
    }
    partial_zero(f)
    partial_zero(g)
    partial(f, Nat.0) = Real.0
    partial(g, Nat.0) = Real.0
    Real.0 <= Real.0
    p(Nat.0)
    forall(k: Nat) {
        if p(k) {
            if forall(i: Nat) { i < k.suc implies f(i) <= g(i) } {
                forall(i: Nat) {
                    if i < k {
                        k < k.suc
                        k <= k.suc
                        lt_and_lte(i, k, k.suc)
                        i < k.suc
                        f(i) <= g(i)
                    }
                }
                p(k) = ((forall(i: Nat) { i < k implies f(i) <= g(i) }) implies partial(f, k) <= partial(g, k))
                partial(f, k) <= partial(g, k)
                k < k.suc
                f(k) <= g(k)
                add_lte_add(partial(f, k), partial(g, k), f(k), g(k))
                partial(f, k) + f(k) <= partial(g, k) + g(k)
                partial_split_last(f, k)
                partial_split_last(g, k)
                partial(f, k.suc) = partial(f, k) + f(k)
                partial(g, k.suc) = partial(g, k) + g(k)
                partial(f, k.suc) <= partial(g, k.suc)
            }
            p(k.suc)
        }
    }
    p(Nat.0) and forall(k: Nat) { p(k) implies p(k.suc) }
    alt_induction(p)
    forall(k: Nat) { p(k) }
    p(n)
}

/// A constant real sequence.
define const_real_fn(c: Real) -> (Nat -> Real) {
    function(i: Nat) { c }
}

/// A sequence shifted down by a constant.
define sub_const_fn(f: Nat -> Real, c: Real) -> (Nat -> Real) {
    function(i: Nat) { f(i) - c }
}

/// A sequence divided pointwise by a constant.
define div_const_fn(f: Nat -> Real, c: Real) -> (Nat -> Real) {
    function(i: Nat) { f(i) / c }
}

/// The pointwise negation of a sequence.
define neg_fn(f: Nat -> Real) -> (Nat -> Real) {
    function(i: Nat) { -f(i) }
}

/// The partial sum of a constant-valued sequence is the count times the value.
theorem partial_const(f: Nat -> Real, n: Nat, c: Real) {
    (forall(i: Nat) { i < n implies f(i) = c }) implies partial(f, n) = from_nat[Real](n) * c
} by {
    define p(k: Nat) -> Bool {
        (forall(i: Nat) { i < k implies f(i) = c }) implies partial(f, k) = from_nat[Real](k) * c
    }
    partial_zero(f)
    partial(f, Nat.0) = Real.0
    from_nat_zero[Real]
    from_nat[Real](Nat.0) = Real.0
    mul_zero_left(c)
    Real.0 * c = Real.0
    p(Nat.0)
    forall(k: Nat) {
        if p(k) {
            if forall(i: Nat) { i < k.suc implies f(i) = c } {
                forall(i: Nat) {
                    if i < k {
                        k < k.suc
                        k <= k.suc
                        lt_and_lte(i, k, k.suc)
                        i < k.suc
                        f(i) = c
                    }
                }
                p(k) = ((forall(i: Nat) { i < k implies f(i) = c }) implies partial(f, k) = from_nat[Real](k) * c)
                partial(f, k) = from_nat[Real](k) * c
                k < k.suc
                f(k) = c
                partial_split_last(f, k)
                partial(f, k.suc) = partial(f, k) + f(k)
                from_nat_add[Real](k, Nat.1)
                from_nat[Real](k + Nat.1) = from_nat[Real](k) + from_nat[Real](Nat.1)
                k + Nat.1 = k.suc
                from_nat[Real](k.suc) = from_nat[Real](k) + from_nat[Real](Nat.1)
                from_nat_one[Real]
                from_nat[Real](Nat.1) = Real.1
                from_nat[Real](k.suc) = from_nat[Real](k) + Real.1
                mul_distrib_right(from_nat[Real](k), Real.1, c)
                (from_nat[Real](k) + Real.1) * c = from_nat[Real](k) * c + Real.1 * c
                mul_one_left(c)
                Real.1 * c = c
                (from_nat[Real](k) + Real.1) * c = from_nat[Real](k) * c + c
                from_nat[Real](k) * c + c = (from_nat[Real](k) + Real.1) * c
                from_nat[Real](k.suc) * c = (from_nat[Real](k) + Real.1) * c
                partial(f, k) + f(k) = from_nat[Real](k) * c + c
                partial(f, k.suc) = from_nat[Real](k.suc) * c
            }
            p(k.suc)
        }
    }
    p(Nat.0) and forall(k: Nat) { p(k) implies p(k.suc) }
    alt_induction(p)
    forall(k: Nat) { p(k) }
    p(n)
}

/// The partial sum of a sequence shifted by a constant is the partial sum
/// shifted by the count times the constant.
theorem partial_sub_const(f: Nat -> Real, c: Real, n: Nat) {
    partial(sub_const_fn(f, c), n) = partial(f, n) - from_nat[Real](n) * c
} by {
    forall(i: Nat) {
        if i < n {
            add_fn(f, const_real_fn(-c))(i) = f(i) + const_real_fn(-c)(i)
            const_real_fn(-c)(i) = -c
            add_fn(f, const_real_fn(-c))(i) = f(i) + -c
            sub_const_fn(f, c)(i) = f(i) - c
            f(i) - c = f(i) + -c
            add_fn(f, const_real_fn(-c))(i) = sub_const_fn(f, c)(i)
        }
    }
    partial_pointwise_eq(add_fn(f, const_real_fn(-c)), sub_const_fn(f, c), n)
    partial(add_fn(f, const_real_fn(-c)), n) = partial(sub_const_fn(f, c), n)
    partial_add(f, const_real_fn(-c), n)
    partial(f, n) + partial(const_real_fn(-c), n) = partial(add_fn(f, const_real_fn(-c)), n)
    partial_const(const_real_fn(-c), n, -c)
    partial(const_real_fn(-c), n) = from_nat[Real](n) * -c
    mul_neg_right(from_nat[Real](n), c)
    from_nat[Real](n) * -c = -(from_nat[Real](n) * c)
    partial(f, n) + partial(const_real_fn(-c), n) = partial(f, n) + -(from_nat[Real](n) * c)
    partial(f, n) + -(from_nat[Real](n) * c) = partial(f, n) - from_nat[Real](n) * c
    partial(sub_const_fn(f, c), n) = partial(f, n) - from_nat[Real](n) * c
}

/// The partial sum of a sequence divided by a nonzero constant is the partial
/// sum divided by the constant.
theorem partial_div_const(f: Nat -> Real, c: Real, n: Nat) {
    c != Real.0 implies partial(div_const_fn(f, c), n) = partial(f, n) / c
} by {
    if c != Real.0 {
        forall(i: Nat) {
            if i < n {
                mul_fn(Real.1 / c, f)(i) = (Real.1 / c) * f(i)
                div_const_fn(f, c)(i) = f(i) / c
                mul_one_over(f(i), c)
                f(i) * (Real.1 / c) = f(i) / c
                (Real.1 / c) * f(i) = f(i) * (Real.1 / c)
                mul_fn(Real.1 / c, f)(i) = div_const_fn(f, c)(i)
            }
        }
        partial_pointwise_eq(mul_fn(Real.1 / c, f), div_const_fn(f, c), n)
        partial(mul_fn(Real.1 / c, f), n) = partial(div_const_fn(f, c), n)
        partial_scalar_mul(Real.1 / c, f, n)
        (Real.1 / c) * partial(f, n) = partial(mul_fn(Real.1 / c, f), n)
        (Real.1 / c) * partial(f, n) = partial(div_const_fn(f, c), n)
        mul_one_over(partial(f, n), c)
        partial(f, n) * (Real.1 / c) = partial(f, n) / c
        (Real.1 / c) * partial(f, n) = partial(f, n) * (Real.1 / c)
        partial(div_const_fn(f, c), n) = partial(f, n) / c
    }
}

/// Negating a sequence negates its partial sums.
theorem partial_neg(f: Nat -> Real, n: Nat) {
    partial(neg_fn(f), n) = -partial(f, n)
} by {
    forall(i: Nat) {
        if i < n {
            mul_fn(-Real.1, f)(i) = (-Real.1) * f(i)
            neg_fn(f)(i) = -f(i)
            mul_neg_one_left(f(i))
            (-Real.1) * f(i) = -f(i)
            mul_fn(-Real.1, f)(i) = neg_fn(f)(i)
        }
    }
    partial_pointwise_eq(mul_fn(-Real.1, f), neg_fn(f), n)
    partial(mul_fn(-Real.1, f), n) = partial(neg_fn(f), n)
    partial_scalar_mul(-Real.1, f, n)
    (-Real.1) * partial(f, n) = partial(mul_fn(-Real.1, f), n)
    (-Real.1) * partial(f, n) = partial(neg_fn(f), n)
    mul_neg_one_left(partial(f, n))
    (-Real.1) * partial(f, n) = -partial(f, n)
    partial(neg_fn(f), n) = -partial(f, n)
}

/// The exponential lies strictly above its tangent line away from zero.
theorem exp_gt_one_plus_x_of_ne_zero(x: Real) {
    x != Real.0 implies x.exp > Real.1 + x
} by {
    if not x.exp > Real.1 + x {
        if x > Real.0 {
            exp_gt_one_plus_x(x)
            x.exp > Real.1 + x
            false
        } else {
            if x < Real.0 {
                            let t = -x
                            lt_add_right(x, Real.0, -x)
                            x + -x < Real.0 + -x
                            Real.0 + -x = -x
                            t > Real.0
                            if t >= Real.1 {
                                lte_add_right(Real.1, t, -t)
                                Real.1 + -t <= t + -t
                                add_neg_eq_zero(t)
                                t + -t = Real.0
                                Real.1 - t <= Real.0
                                exp_pos(-t)
                                (-t).exp > Real.0
                                lt_of_lte_of_lt(Real.1 - t, Real.0, (-t).exp)
                                Real.1 - t < (-t).exp
                                (-t).exp > Real.1 - t
                                -t = x
                                Real.1 - t = Real.1 + x
                                x.exp > Real.1 + x
                                false
                            } else {
                                not_gte_imp_lt(t, Real.1)
                                t < Real.1
                                let h = Real.one_half * t
                                one_half_positive
                                Real.one_half > Real.0
                                gt_zero_imp_pos(Real.one_half)
                                gt_zero_imp_pos(t)
                                mul_pos_pos(Real.one_half, t)
                                (Real.one_half * t).is_positive
                                pos_gt_zero(Real.one_half * t)
                                h > Real.0
                                one_half_plus_one_half
                                Real.one_half + Real.one_half = Real.1
                                lt_add_left(Real.0, Real.one_half, Real.one_half)
                                Real.one_half + Real.0 < Real.one_half + Real.one_half
                                Real.one_half + Real.0 = Real.one_half
                                Real.one_half < Real.1
                                lt_mul_pos_right(Real.one_half, Real.1, t)
                                Real.one_half * t < Real.1 * t
                                mul_one_left(t)
                                Real.1 * t = t
                                Real.one_half * t < t
                                lt_trans(Real.one_half * t, t, Real.1)
                                Real.one_half * t < Real.1
                                h < Real.1
                                lt_add_right(h, Real.1, -h)
                                h + -h < Real.1 + -h
                                add_neg_eq_zero(h)
                                h + -h = Real.0
                                Real.1 - h > Real.0
                                real_one_div_pos(Real.1 - h)
                                Real.1 / (Real.1 - h) > Real.0
                                h >= Real.0
                                exp_le_geometric_recip(h)
                                h.exp <= Real.1 / (Real.1 - h)
                                exp_add(h, h)
                                (h + h).exp = h.exp * h.exp
                                mul_distrib_right(Real.one_half, Real.one_half, t)
                                (Real.one_half + Real.one_half) * t = Real.one_half * t + Real.one_half * t
                                Real.1 * t = (Real.one_half + Real.one_half) * t
                                Real.one_half * t + Real.one_half * t = Real.1 * t
                                Real.one_half * t + Real.one_half * t = t
                                h + h = t
                                t.exp = h.exp * h.exp
                                exp_pos(h)
                                h.exp > Real.0
                                h.exp >= Real.0
                                Real.1 / (Real.1 - h) >= Real.0
                                mul_le_mul_nonneg(h.exp, h.exp, Real.1 / (Real.1 - h), Real.1 / (Real.1 - h))
                                h.exp * h.exp <= (Real.1 / (Real.1 - h)) * (Real.1 / (Real.1 - h))
                                t.exp <= (Real.1 / (Real.1 - h)) * (Real.1 / (Real.1 - h))
                                real_square_add_expanded(Real.1, -h)
                                (Real.1 + -h) * (Real.1 + -h) =
                                    Real.1 * Real.1 + Real.1 * -h + (Real.1 * -h + (-h) * (-h))
                                Real.1 + -h = Real.1 - h
                                Real.1 * Real.1 = Real.1
                                mul_one_left(-h)
                                Real.1 * -h = -h
                                mul_neg_left(h, -h)
                                -h * -h = -(h * -h)
                                mul_neg_right(h, h)
                                h * -h = -(h * h)
                                -(h * -h) = --(h * h)
                                neg_neg(h * h)
                                --(h * h) = h * h
                                -h * -h = h * h
                                (Real.1 - h) * (Real.1 - h) = Real.1 + -h + (-h + h * h)
                                Real.1 + -h + (-h + h * h) = Real.1 + (-h + -h) + h * h
                                -h + -h = -(h + h)
                                h + h = t
                                -(h + h) = -t
                                Real.1 + (-h + -h) + h * h = Real.1 + -t + h * h
                                Real.1 + -t = Real.1 - t
                                (Real.1 - h) * (Real.1 - h) = Real.1 - t + h * h
                                gt_zero_imp_pos(h)
                                mul_pos_pos(h, h)
                                (h * h).is_positive
                                pos_gt_zero(h * h)
                                h * h > Real.0
                                lt_add_left(h * h, Real.0, Real.1 - t)
                                (Real.1 - t) + Real.0 < (Real.1 - t) + h * h
                                (Real.1 - t) + Real.0 = Real.1 - t
                                Real.1 - t < Real.1 - t + h * h
                                Real.1 - t < (Real.1 - h) * (Real.1 - h)
                                lt_add_right(t, Real.1, -t)
                                t + -t < Real.1 + -t
                                add_neg_eq_zero(t)
                                t + -t = Real.0
                                Real.1 - t > Real.0
                                gt_zero_imp_pos(Real.1 - t)
                                real_inverse_antitone_pos_strict(Real.1 - t, (Real.1 - h) * (Real.1 - h))
                                ((Real.1 - h) * (Real.1 - h)).inverse < (Real.1 - t).inverse
                                Real.1 / ((Real.1 - h) * (Real.1 - h)) = ((Real.1 - h) * (Real.1 - h)).inverse
                                Real.1 / (Real.1 - t) = (Real.1 - t).inverse
                                Real.1 / ((Real.1 - h) * (Real.1 - h)) < Real.1 / (Real.1 - t)
                                Real.1 - h != Real.0
                                mul_div(Real.1, Real.1 - h, Real.1, Real.1 - h)
                                (Real.1 / (Real.1 - h)) * (Real.1 / (Real.1 - h)) =
                                    (Real.1 * Real.1) / ((Real.1 - h) * (Real.1 - h))
                                Real.1 * Real.1 = Real.1
                                (Real.1 / (Real.1 - h)) * (Real.1 / (Real.1 - h)) =
                                    Real.1 / ((Real.1 - h) * (Real.1 - h))
                                t.exp <= Real.1 / ((Real.1 - h) * (Real.1 - h))
                                lt_of_lte_of_lt(t.exp, Real.1 / ((Real.1 - h) * (Real.1 - h)), Real.1 / (Real.1 - t))
                                t.exp < Real.1 / (Real.1 - t)
                                gt_zero_imp_pos(t.exp)
                                real_inverse_antitone_pos_strict(t.exp, Real.1 / (Real.1 - t))
                                (Real.1 / (Real.1 - t)).inverse < t.exp.inverse
                                Real.1 - t != Real.0
                                one_div_one_div(Real.1 - t)
                                Real.1 / (Real.1 / (Real.1 - t)) = Real.1 - t
                                Real.1 / (Real.1 - t) = (Real.1 - t).inverse
                                (Real.1 / (Real.1 - t)).inverse = Real.1 - t
                                exp_neg(t)
                                (-t).exp = Real.1 / t.exp
                                t.exp.inverse = Real.1 / t.exp
                                Real.1 - t < (-t).exp
                                (-t).exp > Real.1 - t
                                -t = x
                                Real.1 - t = Real.1 + x
                                 x.exp > Real.1 + x
                                 false
                            }
            } else {
                not_gt_imp_lte(x, Real.0)
                x <= Real.0
                not_lt_imp_gte(x, Real.0)
                x >= Real.0
                Real.0 <= x
                lte_antisymm(x, Real.0)
                x = Real.0
                x != Real.0
                false
            }
        }
    }
}

/// If a positive real has logarithm `x - 1`, then it is one.
theorem log_lt_sub_one(x: Real, y: Real) {
    x > Real.0 and x != Real.1 and x.log = Option.some(y) implies y < x - Real.1
} by {
    if x > Real.0 and x != Real.1 and x.log = Option.some(y) {
        log_ne_zero_of_pos_ne_one(x, y)
        y != Real.0
        exp_gt_one_plus_x_of_ne_zero(y)
        y.exp > Real.1 + y
        log_some_of_pos_exists(x)
        let lx: Real satisfy {
            x.log = Option.some(lx)
        }
        exp_log_or_zero(x, lx)
        lx.exp = x
        option_get_or_else_some[Real](lx, Real.0)
        option_get_or_else(Option.some(lx), Real.0) = lx
        x.log.get_or_else(Real.0) = lx
        option_get_or_else_some[Real](y, Real.0)
        option_get_or_else(Option.some(y), Real.0) = y
        x.log.get_or_else(Real.0) = y
        (x.log.get_or_else(Real.0)).exp = x
        y.exp = x
        x > Real.1 + y
        lt_add_right(Real.1 + y, x, -Real.1)
        Real.1 + y + -Real.1 < x + -Real.1
        Real.1 + y + -Real.1 = y + Real.1 + -Real.1
        y + Real.1 + -Real.1 = y + (Real.1 + -Real.1)
        Real.1 + -Real.1 = Real.0
        y + (Real.1 + -Real.1) = y + Real.0
        y + Real.0 = y
        Real.1 + y + -Real.1 = y
        x + -Real.1 = x - Real.1
        y < x - Real.1
    }
}

/// Two nonnegative reals summing to zero are both zero.
theorem add_nonneg_eq_zero_left(a: Real, b: Real) {
    a >= Real.0 and b >= Real.0 and a + b = Real.0 implies a = Real.0
} by {
    if a >= Real.0 and b >= Real.0 and a + b = Real.0 {
        (a + b) + -b = Real.0 + -b
        a + (b + -b) = Real.0 + -b
        add_neg_eq_zero(b)
        b + -b = Real.0
        a + Real.0 = Real.0 + -b
        a = Real.0 + -b
        Real.0 + -b = -b
        a = -b
        lte_add_right(Real.0, b, -b)
        Real.0 + -b <= b + -b
        b + -b = Real.0
        Real.0 + -b <= Real.0
        -b <= Real.0
        a <= Real.0
        Real.0 <= a
        lte_antisymm(a, Real.0)
        a = Real.0
    }
}

/// Two nonnegative reals summing to zero are both zero.
theorem add_nonneg_eq_zero_right(a: Real, b: Real) {
    a >= Real.0 and b >= Real.0 and a + b = Real.0 implies b = Real.0
} by {
    if a >= Real.0 and b >= Real.0 and a + b = Real.0 {
        (a + b) + -a = Real.0 + -a
        b + (a + -a) = Real.0 + -a
        add_neg_eq_zero(a)
        a + -a = Real.0
        b + Real.0 = Real.0 + -a
        b = Real.0 + -a
        Real.0 + -a = -a
        b = -a
        lte_add_right(Real.0, a, -a)
        Real.0 + -a <= a + -a
        a + -a = Real.0
        Real.0 + -a <= Real.0
        -a <= Real.0
        b <= Real.0
        Real.0 <= b
        lte_antisymm(b, Real.0)
        b = Real.0
    }
}

/// A zero partial sum of nonnegative terms has all terms zero.
theorem partial_nonneg_eq_zero_imp_each(f: Nat -> Real, n: Nat) {
    (forall(i: Nat) { i < n implies f(i) >= Real.0 }) and partial(f, n) = Real.0
    implies forall(i: Nat) { i < n implies f(i) = Real.0 }
} by {
    define p(k: Nat) -> Bool {
        nonnegative_on(f, k) implies
            (partial(f, k) = Real.0 implies forall(i: Nat) { i < k implies f(i) = Real.0 })
    }
    forall(i: Nat) {
        if i < Nat.0 {
            not_lt_zero(i)
            false
        }
    }
    p(Nat.0)
    forall(k: Nat) {
        if p(k) {
            if (forall(i: Nat) { i < k.suc implies f(i) >= Real.0 }) and partial(f, k.suc) = Real.0 {
                partial_split_last(f, k)
                partial(f, k.suc) = partial(f, k) + f(k)
                partial(f, k) + f(k) = Real.0
                forall(i: Nat) {
                    if i < k {
                        k < k.suc
                        k <= k.suc
                        lt_and_lte(i, k, k.suc)
                        i < k.suc
                        f(i) >= Real.0
                    }
                }
                partial_nonneg_bounded(f, k)
                nonnegative_on(f, k) = forall(i: Nat) { i < k implies f(i) >= Real.0 }
                partial(f, k) >= Real.0
                k < k.suc
                f(k) >= Real.0
                add_nonneg_eq_zero_left(partial(f, k), f(k))
                partial(f, k) = Real.0
                add_nonneg_eq_zero_right(partial(f, k), f(k))
                f(k) = Real.0
                p(k) = (nonnegative_on(f, k) implies
                    (partial(f, k) = Real.0 implies forall(i: Nat) { i < k implies f(i) = Real.0 }))
                (nonnegative_on(f, k) implies
                    (partial(f, k) = Real.0 implies forall(i: Nat) { i < k implies f(i) = Real.0 }))
                (partial(f, k) = Real.0 implies forall(i: Nat) { i < k implies f(i) = Real.0 })
                nonnegative_on(f, k)
                partial(f, k) = Real.0
                forall(i: Nat) { i < k implies f(i) = Real.0 }
                forall(i: Nat) {
                    if i < k.suc {
                        lt_suc_right(i, k)
                        i = k or i < k
                        if i = k {
                            f(i) = Real.0
                        } else {
                            i < k
                            f(i) = Real.0
                        }
                    }
                }
            }
            p(k.suc)
        }
    }
    p(Nat.0) and forall(k: Nat) { p(k) implies p(k.suc) }
    alt_induction(p)
    forall(k: Nat) { p(k) }
    p(n)
}

/// The logarithm of a positive natural power is the count times the logarithm.
theorem log_pow(c: Real, n: Nat) {
    c > Real.0 implies (c.pow(n)).log = Option.some(from_nat[Real](n) * c.log.get_or_else(Real.0))
} by {
    if c > Real.0 {
        rpow_nat(c, n)
        c.rpow(from_nat[Real](n)) = Option.some(c.pow(n))
        c.rpow(from_nat[Real](n)) = Option.some((from_nat[Real](n) * c.log.get_or_else(Real.0)).exp)
        Option.some(c.pow(n)) = Option.some((from_nat[Real](n) * c.log.get_or_else(Real.0)).exp)
        some_injective[Real](c.pow(n), (from_nat[Real](n) * c.log.get_or_else(Real.0)).exp)
        c.pow(n) = (from_nat[Real](n) * c.log.get_or_else(Real.0)).exp
        log_exp(from_nat[Real](n) * c.log.get_or_else(Real.0))
        ((from_nat[Real](n) * c.log.get_or_else(Real.0)).exp).log = Option.some(from_nat[Real](n) * c.log.get_or_else(Real.0))
        (c.pow(n)).log = Option.some(from_nat[Real](n) * c.log.get_or_else(Real.0))
    }
}

/// The reciprocal power of a positive natural power recovers the base.
theorem rpow_pow_recip(c: Real, n: Nat) {
    c > Real.0 and n != Nat.0 implies (c.pow(n)).rpow(Real.1 / from_nat[Real](n)) = Option.some(c)
} by {
    if c > Real.0 and n != Nat.0 {
        from_nat_real_ne_zero_of_ne_zero(n)
        from_nat[Real](n) != Real.0
        pow_pos(c, n)
        c.pow(n) > Real.0
        (c.pow(n)).rpow(Real.1 / from_nat[Real](n)) =
            Option.some(((Real.1 / from_nat[Real](n)) * (c.pow(n)).log.get_or_else(Real.0)).exp)
        log_pow(c, n)
        (c.pow(n)).log = Option.some(from_nat[Real](n) * c.log.get_or_else(Real.0))
        option_get_or_else_some[Real](from_nat[Real](n) * c.log.get_or_else(Real.0), Real.0)
        option_get_or_else(Option.some(from_nat[Real](n) * c.log.get_or_else(Real.0)), Real.0) =
            from_nat[Real](n) * c.log.get_or_else(Real.0)
        (c.pow(n)).log.get_or_else(Real.0) = from_nat[Real](n) * c.log.get_or_else(Real.0)
        (Real.1 / from_nat[Real](n)) * (c.pow(n)).log.get_or_else(Real.0) =
            (Real.1 / from_nat[Real](n)) * (from_nat[Real](n) * c.log.get_or_else(Real.0))
        (Real.1 / from_nat[Real](n)) * (from_nat[Real](n) * c.log.get_or_else(Real.0)) =
            ((Real.1 / from_nat[Real](n)) * from_nat[Real](n)) * c.log.get_or_else(Real.0)
        mul_div_cancel(Real.1, from_nat[Real](n))
        from_nat[Real](n) * (Real.1 / from_nat[Real](n)) = Real.1
        (Real.1 / from_nat[Real](n)) * from_nat[Real](n) = Real.1
        mul_one_left(c.log.get_or_else(Real.0))
        Real.1 * c.log.get_or_else(Real.0) = c.log.get_or_else(Real.0)
        ((Real.1 / from_nat[Real](n)) * from_nat[Real](n)) * c.log.get_or_else(Real.0) = c.log.get_or_else(Real.0)
        (Real.1 / from_nat[Real](n)) * (c.pow(n)).log.get_or_else(Real.0) = c.log.get_or_else(Real.0)
        (c.pow(n)).rpow(Real.1 / from_nat[Real](n)) = Option.some((c.log.get_or_else(Real.0)).exp)
        log_some_of_pos_exists(c)
        let lc: Real satisfy {
            c.log = Option.some(lc)
        }
        exp_log_or_zero(c, lc)
        lc.exp = c
        option_get_or_else_some[Real](lc, Real.0)
        option_get_or_else(Option.some(lc), Real.0) = lc
        c.log.get_or_else(Real.0) = lc
        (c.log.get_or_else(Real.0)).exp = c
        (c.pow(n)).rpow(Real.1 / from_nat[Real](n)) = Option.some(c)
    }
}

/// A constant-valued finite product is the value raised to the count.
theorem finite_real_product_const(f: Nat -> Real, n: Nat, c: Real) {
    (forall(i: Nat) { i < n implies f(i) = c }) implies finite_real_product(f, n) = c.pow(n)
} by {
    define p(k: Nat) -> Bool {
        (forall(i: Nat) { i < k implies f(i) = c }) implies finite_real_product(f, k) = c.pow(k)
    }
    finite_real_product_zero(f)
    finite_real_product(f, Nat.0) = Real.1
    c.pow(Nat.0) = Real.1
    p(Nat.0)
    forall(k: Nat) {
        if p(k) {
            if forall(i: Nat) { i < k.suc implies f(i) = c } {
                forall(i: Nat) {
                    if i < k {
                        k < k.suc
                        k <= k.suc
                        lt_and_lte(i, k, k.suc)
                        i < k.suc
                        f(i) = c
                    }
                }
                p(k) = ((forall(i: Nat) { i < k implies f(i) = c }) implies finite_real_product(f, k) = c.pow(k))
                finite_real_product(f, k) = c.pow(k)
                k < k.suc
                f(k) = c
                finite_real_product_suc(f, k)
                finite_real_product(f, k.suc) = finite_real_product(f, k) * f(k)
                finite_real_product(f, k) * f(k) = c.pow(k) * c
                pow_suc(c, k)
                c.pow(k.suc) = c * c.pow(k)
                real_mul_comm(c.pow(k), c)
                c.pow(k) * c = c * c.pow(k)
                c.pow(k) * c = c.pow(k.suc)
                finite_real_product(f, k.suc) = c.pow(k.suc)
            }
            p(k.suc)
        }
    }
    p(Nat.0) and forall(k: Nat) { p(k) implies p(k.suc) }
    alt_induction(p)
    forall(k: Nat) { p(k) }
    p(n)
}

/// The arithmetic-geometric mean inequality for positive values: the geometric
/// mean of the first `n` values is at most their arithmetic mean.
theorem am_gm_pos(f: Nat -> Real, n: Nat) {
    n != Nat.0 and positive_on(f, n) implies exists(g: Real) {
        (finite_real_product(f, n)).rpow(Real.1 / from_nat[Real](n)) = Option.some(g)
        and g <= finite_real_mean(f, n)
    }
} by {
    if n != Nat.0 and positive_on(f, n) {
        finite_real_product_pos(f, n)
        finite_real_product(f, n) > Real.0
        partial_pos_bounded(f, n)
        partial(f, n) > Real.0
        from_nat_real_pos_of_ne_zero(n)
        from_nat[Real](n) > Real.0
        div_pos_of_pos_pos(partial(f, n), from_nat[Real](n))
        partial(f, n) / from_nat[Real](n) > Real.0
        finite_real_mean(f, n) = partial(f, n) / from_nat[Real](n)
        finite_real_mean(f, n) > Real.0
        log_finite_product(f, n)
        (finite_real_product(f, n)).log = Option.some(partial(log_value_fn(f), n))
        option_get_or_else_some[Real](partial(log_value_fn(f), n), Real.0)
        option_get_or_else(Option.some(partial(log_value_fn(f), n)), Real.0) = partial(log_value_fn(f), n)
        (finite_real_product(f, n)).log.get_or_else(Real.0) = partial(log_value_fn(f), n)
        forall(i: Nat) {
            if i < n {
                positive_on(f, n) = forall(j: Nat) { j < n implies f(j) > Real.0 }
                f(i) > Real.0
                div_pos_of_pos_pos(f(i), finite_real_mean(f, n))
                f(i) / finite_real_mean(f, n) > Real.0
                log_div(f(i), finite_real_mean(f, n))
                (f(i) / finite_real_mean(f, n)).log =
                    Option.some((f(i)).log.get_or_else(Real.0) - (finite_real_mean(f, n)).log.get_or_else(Real.0))
                log_le_sub_one(f(i) / finite_real_mean(f, n),
                    (f(i)).log.get_or_else(Real.0) - (finite_real_mean(f, n)).log.get_or_else(Real.0))
                (f(i)).log.get_or_else(Real.0) - (finite_real_mean(f, n)).log.get_or_else(Real.0) <= f(i) / finite_real_mean(f, n) - Real.1
                sub_const_fn(log_value_fn(f), (finite_real_mean(f, n)).log.get_or_else(Real.0))(i) =
                    (f(i)).log.get_or_else(Real.0) - (finite_real_mean(f, n)).log.get_or_else(Real.0)
                sub_const_fn(div_const_fn(f, finite_real_mean(f, n)), Real.1)(i) =
                    f(i) / finite_real_mean(f, n) - Real.1
                sub_const_fn(log_value_fn(f), (finite_real_mean(f, n)).log.get_or_else(Real.0))(i) <= sub_const_fn(div_const_fn(f, finite_real_mean(f, n)), Real.1)(i)
            }
        }
        partial_le_range(sub_const_fn(log_value_fn(f), (finite_real_mean(f, n)).log.get_or_else(Real.0)),
            sub_const_fn(div_const_fn(f, finite_real_mean(f, n)), Real.1), n)
        partial(sub_const_fn(log_value_fn(f), (finite_real_mean(f, n)).log.get_or_else(Real.0)), n) <= partial(sub_const_fn(div_const_fn(f, finite_real_mean(f, n)), Real.1), n)
        partial_sub_const(log_value_fn(f), (finite_real_mean(f, n)).log.get_or_else(Real.0), n)
        partial(sub_const_fn(log_value_fn(f), (finite_real_mean(f, n)).log.get_or_else(Real.0)), n) =
            partial(log_value_fn(f), n) - from_nat[Real](n) * (finite_real_mean(f, n)).log.get_or_else(Real.0)
        partial_sub_const(div_const_fn(f, finite_real_mean(f, n)), Real.1, n)
        partial(sub_const_fn(div_const_fn(f, finite_real_mean(f, n)), Real.1), n) =
            partial(div_const_fn(f, finite_real_mean(f, n)), n) - from_nat[Real](n) * Real.1
        finite_real_mean(f, n) != Real.0
        partial_div_const(f, finite_real_mean(f, n), n)
        partial(div_const_fn(f, finite_real_mean(f, n)), n) = partial(f, n) / finite_real_mean(f, n)
        partial(sub_const_fn(div_const_fn(f, finite_real_mean(f, n)), Real.1), n) =
            partial(f, n) / finite_real_mean(f, n) - from_nat[Real](n) * Real.1
        mul_one_right(from_nat[Real](n))
        from_nat[Real](n) * Real.1 = from_nat[Real](n)
        partial(sub_const_fn(div_const_fn(f, finite_real_mean(f, n)), Real.1), n) =
            partial(f, n) / finite_real_mean(f, n) - from_nat[Real](n)
        finite_real_mean_mul_count(f, n)
        finite_real_mean(f, n) * from_nat[Real](n) = partial(f, n)
        real_mul_comm(finite_real_mean(f, n), from_nat[Real](n))
        from_nat[Real](n) * finite_real_mean(f, n) = partial(f, n)
        mul_frac_assoc(from_nat[Real](n), finite_real_mean(f, n), finite_real_mean(f, n))
        from_nat[Real](n) * (finite_real_mean(f, n) / finite_real_mean(f, n)) =
            (from_nat[Real](n) * finite_real_mean(f, n)) / finite_real_mean(f, n)
        mul_inverse(finite_real_mean(f, n))
        finite_real_mean(f, n) * finite_real_mean(f, n).inverse = Real.1
        finite_real_mean(f, n) / finite_real_mean(f, n) = Real.1
        mul_one_right(from_nat[Real](n))
        from_nat[Real](n) * Real.1 = from_nat[Real](n)
        from_nat[Real](n) * (finite_real_mean(f, n) / finite_real_mean(f, n)) = from_nat[Real](n)
        (from_nat[Real](n) * finite_real_mean(f, n)) / finite_real_mean(f, n) = from_nat[Real](n)
        partial(f, n) / finite_real_mean(f, n) = from_nat[Real](n)
        partial(sub_const_fn(div_const_fn(f, finite_real_mean(f, n)), Real.1), n) =
            from_nat[Real](n) - from_nat[Real](n)
        from_nat[Real](n) + -from_nat[Real](n) = Real.0
        add_neg_eq_zero(from_nat[Real](n))
        from_nat[Real](n) - from_nat[Real](n) = Real.0
        partial(sub_const_fn(div_const_fn(f, finite_real_mean(f, n)), Real.1), n) = Real.0
        partial(log_value_fn(f), n) - from_nat[Real](n) * (finite_real_mean(f, n)).log.get_or_else(Real.0) <= Real.0
        lte_add_right(partial(log_value_fn(f), n) + -(from_nat[Real](n) * (finite_real_mean(f, n)).log.get_or_else(Real.0)), Real.0, from_nat[Real](n) * (finite_real_mean(f, n)).log.get_or_else(Real.0))
        partial(log_value_fn(f), n) + -(from_nat[Real](n) * (finite_real_mean(f, n)).log.get_or_else(Real.0)) +
            from_nat[Real](n) * (finite_real_mean(f, n)).log.get_or_else(Real.0) <= Real.0 + from_nat[Real](n) * (finite_real_mean(f, n)).log.get_or_else(Real.0)
        add_neg_eq_zero(from_nat[Real](n) * (finite_real_mean(f, n)).log.get_or_else(Real.0))
        from_nat[Real](n) * (finite_real_mean(f, n)).log.get_or_else(Real.0) +
            -(from_nat[Real](n) * (finite_real_mean(f, n)).log.get_or_else(Real.0)) = Real.0
        partial(log_value_fn(f), n) <= from_nat[Real](n) * (finite_real_mean(f, n)).log.get_or_else(Real.0)
        from_nat_real_ne_zero_of_ne_zero(n)
        from_nat[Real](n) != Real.0
        real_one_div_pos(from_nat[Real](n))
        Real.1 / from_nat[Real](n) > Real.0
        lte_mul_nonneg_left(partial(log_value_fn(f), n),
            from_nat[Real](n) * (finite_real_mean(f, n)).log.get_or_else(Real.0), Real.1 / from_nat[Real](n))
        (Real.1 / from_nat[Real](n)) * partial(log_value_fn(f), n) <= (Real.1 / from_nat[Real](n)) * (from_nat[Real](n) * (finite_real_mean(f, n)).log.get_or_else(Real.0))
        (Real.1 / from_nat[Real](n)) * (from_nat[Real](n) * (finite_real_mean(f, n)).log.get_or_else(Real.0)) =
            ((Real.1 / from_nat[Real](n)) * from_nat[Real](n)) * (finite_real_mean(f, n)).log.get_or_else(Real.0)
        mul_div_cancel(Real.1, from_nat[Real](n))
        from_nat[Real](n) * (Real.1 / from_nat[Real](n)) = Real.1
        (Real.1 / from_nat[Real](n)) * from_nat[Real](n) = Real.1
        mul_one_left((finite_real_mean(f, n)).log.get_or_else(Real.0))
        Real.1 * (finite_real_mean(f, n)).log.get_or_else(Real.0) = (finite_real_mean(f, n)).log.get_or_else(Real.0)
        ((Real.1 / from_nat[Real](n)) * from_nat[Real](n)) * (finite_real_mean(f, n)).log.get_or_else(Real.0) =
            (finite_real_mean(f, n)).log.get_or_else(Real.0)
        (Real.1 / from_nat[Real](n)) * (from_nat[Real](n) * (finite_real_mean(f, n)).log.get_or_else(Real.0)) =
            (finite_real_mean(f, n)).log.get_or_else(Real.0)
        (Real.1 / from_nat[Real](n)) * partial(log_value_fn(f), n) <= (finite_real_mean(f, n)).log.get_or_else(Real.0)
        exp_monotone((Real.1 / from_nat[Real](n)) * partial(log_value_fn(f), n),
            (finite_real_mean(f, n)).log.get_or_else(Real.0))
        ((Real.1 / from_nat[Real](n)) * partial(log_value_fn(f), n)).exp <= ((finite_real_mean(f, n)).log.get_or_else(Real.0)).exp
        log_some_of_pos_exists(finite_real_mean(f, n))
        let lm: Real satisfy {
            (finite_real_mean(f, n)).log = Option.some(lm)
        }
        exp_log_or_zero(finite_real_mean(f, n), lm)
        lm.exp = finite_real_mean(f, n)
        option_get_or_else_some[Real](lm, Real.0)
        option_get_or_else(Option.some(lm), Real.0) = lm
        (finite_real_mean(f, n)).log.get_or_else(Real.0) = lm
        ((finite_real_mean(f, n)).log.get_or_else(Real.0)).exp = finite_real_mean(f, n)
        ((Real.1 / from_nat[Real](n)) * partial(log_value_fn(f), n)).exp <= finite_real_mean(f, n)
        (finite_real_product(f, n)).rpow(Real.1 / from_nat[Real](n)) =
            Option.some(((Real.1 / from_nat[Real](n)) * (finite_real_product(f, n)).log.get_or_else(Real.0)).exp)
        (Real.1 / from_nat[Real](n)) * (finite_real_product(f, n)).log.get_or_else(Real.0) =
            (Real.1 / from_nat[Real](n)) * partial(log_value_fn(f), n)
        ((Real.1 / from_nat[Real](n)) * (finite_real_product(f, n)).log.get_or_else(Real.0)).exp =
            ((Real.1 / from_nat[Real](n)) * partial(log_value_fn(f), n)).exp
        ((Real.1 / from_nat[Real](n)) * (finite_real_product(f, n)).log.get_or_else(Real.0)).exp <= finite_real_mean(f, n)
        exists(g: Real) {
            g = ((Real.1 / from_nat[Real](n)) * (finite_real_product(f, n)).log.get_or_else(Real.0)).exp
            and (finite_real_product(f, n)).rpow(Real.1 / from_nat[Real](n)) = Option.some(g)
            and g <= finite_real_mean(f, n)
        }
    }
}

/// A nonnegative sequence that is not everywhere positive has a zero factor.
theorem not_positive_on_imp_zero_factor(f: Nat -> Real, n: Nat) {
    nonnegative_on(f, n) and not positive_on(f, n) implies exists(i: Nat) {
        i < n and f(i) = Real.0
    }
} by {
    if nonnegative_on(f, n) and not positive_on(f, n) {
        positive_on(f, n) = forall(i: Nat) { i < n implies f(i) > Real.0 }
        not forall(i: Nat) { i < n implies f(i) > Real.0 }
        not_forall_imp_exists_not(function(i: Nat) { i < n implies f(i) > Real.0 })
        exists(i: Nat) { not (i < n implies f(i) > Real.0) }
        let i: Nat satisfy {
            not (i < n implies f(i) > Real.0)
        }
        not_implies(i < n, f(i) > Real.0)
        (not (i < n implies f(i) > Real.0)) = (i < n and not f(i) > Real.0)
        i < n and not f(i) > Real.0
        i < n
        not f(i) > Real.0
        nonnegative_on(f, n) = forall(j: Nat) { j < n implies f(j) >= Real.0 }
        f(i) >= Real.0
        not_gt_imp_lte(f(i), Real.0)
        f(i) <= Real.0
        Real.0 <= f(i)
        lte_antisymm(f(i), Real.0)
        f(i) = Real.0
        exists(k: Nat) {
            k < n and f(k) = Real.0
        }
    }
}

/// A finite product with a zero factor is zero.
theorem finite_real_product_zero_of_factor(f: Nat -> Real, n: Nat, i: Nat) {
    i < n and f(i) = Real.0 implies finite_real_product(f, n) = Real.0
} by {
    define p(k: Nat) -> Bool {
        i < k implies finite_real_product(f, k) = Real.0
    }
    if i < Nat.0 {
        not_lt_zero(i)
        false
    }
    p(Nat.0)
    forall(k: Nat) {
        if p(k) {
            if i < k.suc {
                lt_suc_right(i, k)
                i = k or i < k
                if i = k {
                    f(k) = Real.0
                    finite_real_product_suc(f, k)
                    finite_real_product(f, k.suc) = finite_real_product(f, k) * f(k)
                    mul_zero_right(finite_real_product(f, k))
                    finite_real_product(f, k) * Real.0 = Real.0
                    finite_real_product(f, k.suc) = Real.0
                } else {
                    i < k
                    p(k) = (i < k implies finite_real_product(f, k) = Real.0)
                    finite_real_product(f, k) = Real.0
                    finite_real_product_suc(f, k)
                    finite_real_product(f, k.suc) = finite_real_product(f, k) * f(k)
                    mul_zero_left(f(k))
                    Real.0 * f(k) = Real.0
                    finite_real_product(f, k.suc) = Real.0
                }
            }
            p(k.suc)
        }
    }
    p(Nat.0) and forall(k: Nat) { p(k) implies p(k.suc) }
    alt_induction(p)
    forall(k: Nat) { p(k) }
    p(n)
    p(n) = (i < n implies finite_real_product(f, n) = Real.0)
    i < n and f(i) = Real.0
    finite_real_product(f, n) = Real.0
}

/// The AM-GM inequality when some of the first `n` values are zero.
theorem am_gm_zero(f: Nat -> Real, n: Nat) {
    n != Nat.0 and nonnegative_on(f, n)
    and exists(i: Nat) { i < n and f(i) = Real.0 }
    implies exists(g: Real) {
        (finite_real_product(f, n)).rpow(Real.1 / from_nat[Real](n)) = Option.some(g)
        and g <= finite_real_mean(f, n)
    }
} by {
    if n != Nat.0 and nonnegative_on(f, n) and exists(i: Nat) { i < n and f(i) = Real.0 } {
        let i: Nat satisfy {
            i < n and f(i) = Real.0
        }
        finite_real_product_zero_of_factor(f, n, i)
        finite_real_product(f, n) = Real.0
        from_nat_real_pos_of_ne_zero(n)
        from_nat[Real](n) > Real.0
        real_one_div_pos(from_nat[Real](n))
        Real.1 / from_nat[Real](n) > Real.0
        rpow_zero_base_pos(Real.1 / from_nat[Real](n))
        (Real.0).rpow(Real.1 / from_nat[Real](n)) = Option.some(Real.0)
        (finite_real_product(f, n)).rpow(Real.1 / from_nat[Real](n)) = Option.some(Real.0)
        partial_nonneg_bounded(f, n)
        partial(f, n) >= Real.0
        div_nonneg_of_nonneg_pos(partial(f, n), from_nat[Real](n))
        partial(f, n) / from_nat[Real](n) >= Real.0
        finite_real_mean(f, n) = partial(f, n) / from_nat[Real](n)
        finite_real_mean(f, n) >= Real.0
        Real.0 <= finite_real_mean(f, n)
        exists(g: Real) {
            g = Real.0
            and (finite_real_product(f, n)).rpow(Real.1 / from_nat[Real](n)) = Option.some(g)
            and g <= finite_real_mean(f, n)
        }
    }
}

/// The arithmetic-geometric mean inequality: the geometric mean of the first
/// `n` nonnegative values is at most their arithmetic mean.
theorem am_gm(f: Nat -> Real, n: Nat) {
    n != Nat.0 and nonnegative_on(f, n) implies exists(g: Real) {
        (finite_real_product(f, n)).rpow(Real.1 / from_nat[Real](n)) = Option.some(g)
        and g <= finite_real_mean(f, n)
    }
} by {
    if positive_on(f, n) {
        am_gm_pos(f, n)
        exists(g: Real) {
            (finite_real_product(f, n)).rpow(Real.1 / from_nat[Real](n)) = Option.some(g)
            and g <= finite_real_mean(f, n)
        }
    } else {
        not_positive_on_imp_zero_factor(f, n)
        exists(i: Nat) { i < n and f(i) = Real.0 }
        am_gm_zero(f, n)
        exists(g: Real) {
            (finite_real_product(f, n)).rpow(Real.1 / from_nat[Real](n)) = Option.some(g)
            and g <= finite_real_mean(f, n)
        }
    }
}

/// If all of the first `n` values equal a nonnegative `c`, the geometric mean
/// equals the arithmetic mean.
theorem am_gm_eq_of_all_eq(f: Nat -> Real, n: Nat, c: Real) {
    n != Nat.0 and c >= Real.0 and (forall(i: Nat) { i < n implies f(i) = c })
    implies exists(g: Real) {
        (finite_real_product(f, n)).rpow(Real.1 / from_nat[Real](n)) = Option.some(g)
        and g = finite_real_mean(f, n)
    }
} by {
    if n != Nat.0 and c >= Real.0 and forall(i: Nat) { i < n implies f(i) = c } {
        finite_real_product_const(f, n, c)
        finite_real_product(f, n) = c.pow(n)
        partial_const(f, n, c)
        partial(f, n) = from_nat[Real](n) * c
        if c = Real.0 {
            zero_or_suc(n)
            let k: Nat satisfy {
                n = k.suc
            }
            k < k.suc
            k < n
            forall(i: Nat) { i < n implies f(i) = c }
            f(k) = c
            f(k) = Real.0
            finite_real_product_zero_of_factor(f, n, k)
            finite_real_product(f, n) = Real.0
            from_nat_real_pos_of_ne_zero(n)
            from_nat[Real](n) > Real.0
            real_one_div_pos(from_nat[Real](n))
            Real.1 / from_nat[Real](n) > Real.0
            rpow_zero_base_pos(Real.1 / from_nat[Real](n))
            (Real.0).rpow(Real.1 / from_nat[Real](n)) = Option.some(Real.0)
            (finite_real_product(f, n)).rpow(Real.1 / from_nat[Real](n)) = Option.some(Real.0)
            partial_const(f, n, Real.0)
            partial(f, n) = from_nat[Real](n) * Real.0
            mul_zero_right(from_nat[Real](n))
            from_nat[Real](n) * Real.0 = Real.0
            partial(f, n) = Real.0
            finite_real_mean(f, n) = partial(f, n) / from_nat[Real](n)
            partial(f, n) / from_nat[Real](n) = Real.0 * from_nat[Real](n).inverse
            mul_zero_left(from_nat[Real](n).inverse)
            Real.0 * from_nat[Real](n).inverse = Real.0
            partial(f, n) / from_nat[Real](n) = Real.0
            finite_real_mean(f, n) = Real.0
            exists(g: Real) {
                (finite_real_product(f, n)).rpow(Real.1 / from_nat[Real](n)) = Option.some(g)
                and g = finite_real_mean(f, n)
            }
        } else {
            if not c > Real.0 {
                not_gt_imp_lte(c, Real.0)
                c <= Real.0
                lte_antisymm(Real.0, c)
                Real.0 = c
                c = Real.0
                false
            }
            c > Real.0
            rpow_pow_recip(c, n)
            (c.pow(n)).rpow(Real.1 / from_nat[Real](n)) = Option.some(c)
            (finite_real_product(f, n)).rpow(Real.1 / from_nat[Real](n)) = Option.some(c)
            from_nat_real_ne_zero_of_ne_zero(n)
            from_nat[Real](n) != Real.0
            mul_frac_assoc(from_nat[Real](n), c, from_nat[Real](n))
            from_nat[Real](n) * (c / from_nat[Real](n)) = (from_nat[Real](n) * c) / from_nat[Real](n)
            mul_div_cancel(c, from_nat[Real](n))
            from_nat[Real](n) * (c / from_nat[Real](n)) = c
            (from_nat[Real](n) * c) / from_nat[Real](n) = c
            finite_real_mean(f, n) = partial(f, n) / from_nat[Real](n)
            partial(f, n) / from_nat[Real](n) = (from_nat[Real](n) * c) / from_nat[Real](n)
            finite_real_mean(f, n) = c
            exists(g: Real) {
                (finite_real_product(f, n)).rpow(Real.1 / from_nat[Real](n)) = Option.some(g)
                and g = finite_real_mean(f, n)
            }
        }
    }
}

/// The per-index quantity (f(i)).log - m.log.
define l_fn(f: Nat -> Real, m: Real) -> (Nat -> Real) {
    sub_const_fn(log_value_fn(f), m.log.get_or_else(Real.0))
}

/// The per-index quantity f(i) / m - 1.
define r_fn(f: Nat -> Real, m: Real) -> (Nat -> Real) {
    sub_const_fn(div_const_fn(f, m), Real.1)
}

/// The per-index gap r_fn(i) - l_fn(i).
define d_fn(f: Nat -> Real, m: Real) -> (Nat -> Real) {
    add_fn(r_fn(f, m), neg_fn(l_fn(f, m)))
}

/// Equality in the positive-case AM-GM forces all values to equal the mean.
theorem am_gm_eq_imp_all_eq_pos(f: Nat -> Real, n: Nat) {
    n != Nat.0 and positive_on(f, n)
    and exists(g: Real) {
        (finite_real_product(f, n)).rpow(Real.1 / from_nat[Real](n)) = Option.some(g)
        and g = finite_real_mean(f, n)
    }
    implies forall(i: Nat) { i < n implies f(i) = finite_real_mean(f, n) }
} by {
    if n != Nat.0 and positive_on(f, n)
        and exists(g: Real) {
            (finite_real_product(f, n)).rpow(Real.1 / from_nat[Real](n)) = Option.some(g)
            and g = finite_real_mean(f, n)
        } {
        let g: Real satisfy {
            (finite_real_product(f, n)).rpow(Real.1 / from_nat[Real](n)) = Option.some(g)
            and g = finite_real_mean(f, n)
        }
        finite_real_product_pos(f, n)
        finite_real_product(f, n) > Real.0
        partial_pos_bounded(f, n)
        partial(f, n) > Real.0
        from_nat_real_pos_of_ne_zero(n)
        from_nat[Real](n) > Real.0
        div_pos_of_pos_pos(partial(f, n), from_nat[Real](n))
        finite_real_mean(f, n) = partial(f, n) / from_nat[Real](n)
        finite_real_mean(f, n) > Real.0
        log_finite_product(f, n)
        (finite_real_product(f, n)).log = Option.some(partial(log_value_fn(f), n))
        option_get_or_else_some[Real](partial(log_value_fn(f), n), Real.0)
        option_get_or_else(Option.some(partial(log_value_fn(f), n)), Real.0) = partial(log_value_fn(f), n)
        (finite_real_product(f, n)).log.get_or_else(Real.0) = partial(log_value_fn(f), n)
        (finite_real_product(f, n)).rpow(Real.1 / from_nat[Real](n)) =
            Option.some(((Real.1 / from_nat[Real](n)) * (finite_real_product(f, n)).log.get_or_else(Real.0)).exp)
        Option.some(g) = Option.some(((Real.1 / from_nat[Real](n)) * (finite_real_product(f, n)).log.get_or_else(Real.0)).exp)
        some_injective[Real](g, ((Real.1 / from_nat[Real](n)) * (finite_real_product(f, n)).log.get_or_else(Real.0)).exp)
        g = ((Real.1 / from_nat[Real](n)) * (finite_real_product(f, n)).log.get_or_else(Real.0)).exp
        g = ((Real.1 / from_nat[Real](n)) * partial(log_value_fn(f), n)).exp
        g = finite_real_mean(f, n)
        ((Real.1 / from_nat[Real](n)) * partial(log_value_fn(f), n)).exp = finite_real_mean(f, n)
        log_some_of_pos_exists(finite_real_mean(f, n))
        let lm: Real satisfy {
            (finite_real_mean(f, n)).log = Option.some(lm)
        }
        exp_log_or_zero(finite_real_mean(f, n), lm)
        lm.exp = finite_real_mean(f, n)
        option_get_or_else_some[Real](lm, Real.0)
        option_get_or_else(Option.some(lm), Real.0) = lm
        (finite_real_mean(f, n)).log.get_or_else(Real.0) = lm
        ((finite_real_mean(f, n)).log.get_or_else(Real.0)).exp = finite_real_mean(f, n)
        ((Real.1 / from_nat[Real](n)) * partial(log_value_fn(f), n)).exp =
            ((finite_real_mean(f, n)).log.get_or_else(Real.0)).exp
        exp_injective((Real.1 / from_nat[Real](n)) * partial(log_value_fn(f), n),
            (finite_real_mean(f, n)).log.get_or_else(Real.0))
        (Real.1 / from_nat[Real](n)) * partial(log_value_fn(f), n) = (finite_real_mean(f, n)).log.get_or_else(Real.0)
        from_nat_real_ne_zero_of_ne_zero(n)
        from_nat[Real](n) != Real.0
        from_nat[Real](n) * ((Real.1 / from_nat[Real](n)) * partial(log_value_fn(f), n)) =
            from_nat[Real](n) * (finite_real_mean(f, n)).log.get_or_else(Real.0)
        mul_assoc_real(from_nat[Real](n), Real.1 / from_nat[Real](n), partial(log_value_fn(f), n))
        (from_nat[Real](n) * (Real.1 / from_nat[Real](n))) * partial(log_value_fn(f), n) =
            from_nat[Real](n) * ((Real.1 / from_nat[Real](n)) * partial(log_value_fn(f), n))
        mul_div_cancel(Real.1, from_nat[Real](n))
        from_nat[Real](n) * (Real.1 / from_nat[Real](n)) = Real.1
        (from_nat[Real](n) * (Real.1 / from_nat[Real](n))) * partial(log_value_fn(f), n) =
            partial(log_value_fn(f), n)
        partial(log_value_fn(f), n) = from_nat[Real](n) * (finite_real_mean(f, n)).log.get_or_else(Real.0)
        forall(i: Nat) {
            if i < n {
                positive_on(f, n) = forall(j: Nat) { j < n implies f(j) > Real.0 }
                f(i) > Real.0
                div_pos_of_pos_pos(f(i), finite_real_mean(f, n))
                f(i) / finite_real_mean(f, n) > Real.0
                log_div(f(i), finite_real_mean(f, n))
                (f(i) / finite_real_mean(f, n)).log =
                    Option.some((f(i)).log.get_or_else(Real.0) - (finite_real_mean(f, n)).log.get_or_else(Real.0))
                log_le_sub_one(f(i) / finite_real_mean(f, n),
                    (f(i)).log.get_or_else(Real.0) - (finite_real_mean(f, n)).log.get_or_else(Real.0))
                (f(i)).log.get_or_else(Real.0) - (finite_real_mean(f, n)).log.get_or_else(Real.0) <= f(i) / finite_real_mean(f, n) - Real.1
                l_fn(f, finite_real_mean(f, n))(i) = (f(i)).log.get_or_else(Real.0) - (finite_real_mean(f, n)).log.get_or_else(Real.0)
                r_fn(f, finite_real_mean(f, n))(i) = f(i) / finite_real_mean(f, n) - Real.1
                l_fn(f, finite_real_mean(f, n))(i) <= r_fn(f, finite_real_mean(f, n))(i)
            }
        }
        partial_le_range(l_fn(f, finite_real_mean(f, n)), r_fn(f, finite_real_mean(f, n)), n)
        partial(l_fn(f, finite_real_mean(f, n)), n) <= partial(r_fn(f, finite_real_mean(f, n)), n)
        partial_sub_const(log_value_fn(f), (finite_real_mean(f, n)).log.get_or_else(Real.0), n)
        partial(l_fn(f, finite_real_mean(f, n)), n) =
            partial(log_value_fn(f), n) - from_nat[Real](n) * (finite_real_mean(f, n)).log.get_or_else(Real.0)
        partial(log_value_fn(f), n) - from_nat[Real](n) * (finite_real_mean(f, n)).log.get_or_else(Real.0) = Real.0
        partial(l_fn(f, finite_real_mean(f, n)), n) = Real.0
        partial_sub_const(div_const_fn(f, finite_real_mean(f, n)), Real.1, n)
        partial(r_fn(f, finite_real_mean(f, n)), n) =
            partial(div_const_fn(f, finite_real_mean(f, n)), n) - from_nat[Real](n) * Real.1
        finite_real_mean(f, n) != Real.0
        partial_div_const(f, finite_real_mean(f, n), n)
        partial(div_const_fn(f, finite_real_mean(f, n)), n) = partial(f, n) / finite_real_mean(f, n)
        partial(r_fn(f, finite_real_mean(f, n)), n) =
            partial(f, n) / finite_real_mean(f, n) - from_nat[Real](n) * Real.1
        mul_one_right(from_nat[Real](n))
        from_nat[Real](n) * Real.1 = from_nat[Real](n)
        partial(r_fn(f, finite_real_mean(f, n)), n) =
            partial(f, n) / finite_real_mean(f, n) - from_nat[Real](n)
        finite_real_mean_mul_count(f, n)
        finite_real_mean(f, n) * from_nat[Real](n) = partial(f, n)
        real_mul_comm(finite_real_mean(f, n), from_nat[Real](n))
        from_nat[Real](n) * finite_real_mean(f, n) = partial(f, n)
        mul_frac_assoc(from_nat[Real](n), finite_real_mean(f, n), finite_real_mean(f, n))
        from_nat[Real](n) * (finite_real_mean(f, n) / finite_real_mean(f, n)) =
            (from_nat[Real](n) * finite_real_mean(f, n)) / finite_real_mean(f, n)
        mul_inverse(finite_real_mean(f, n))
        finite_real_mean(f, n) * finite_real_mean(f, n).inverse = Real.1
        finite_real_mean(f, n) / finite_real_mean(f, n) = Real.1
        mul_one_right(from_nat[Real](n))
        from_nat[Real](n) * Real.1 = from_nat[Real](n)
        from_nat[Real](n) * (finite_real_mean(f, n) / finite_real_mean(f, n)) = from_nat[Real](n)
        (from_nat[Real](n) * finite_real_mean(f, n)) / finite_real_mean(f, n) = from_nat[Real](n)
        partial(f, n) / finite_real_mean(f, n) = from_nat[Real](n)
        partial(r_fn(f, finite_real_mean(f, n)), n) = from_nat[Real](n) - from_nat[Real](n)
        from_nat[Real](n) - from_nat[Real](n) = Real.0
        partial(r_fn(f, finite_real_mean(f, n)), n) = Real.0
        partial_add(r_fn(f, finite_real_mean(f, n)), neg_fn(l_fn(f, finite_real_mean(f, n))), n)
        partial(r_fn(f, finite_real_mean(f, n)), n) +
            partial(neg_fn(l_fn(f, finite_real_mean(f, n))), n) =
            partial(d_fn(f, finite_real_mean(f, n)), n)
        partial_neg(l_fn(f, finite_real_mean(f, n)), n)
        partial(neg_fn(l_fn(f, finite_real_mean(f, n))), n) = -partial(l_fn(f, finite_real_mean(f, n)), n)
        partial(d_fn(f, finite_real_mean(f, n)), n) = Real.0 + -Real.0
        Real.0 + -Real.0 = Real.0
        partial(d_fn(f, finite_real_mean(f, n)), n) = Real.0
        forall(i: Nat) {
            if i < n {
                positive_on(f, n) = forall(j: Nat) { j < n implies f(j) > Real.0 }
                f(i) > Real.0
                div_pos_of_pos_pos(f(i), finite_real_mean(f, n))
                f(i) / finite_real_mean(f, n) > Real.0
                log_div(f(i), finite_real_mean(f, n))
                (f(i) / finite_real_mean(f, n)).log =
                    Option.some((f(i)).log.get_or_else(Real.0) - (finite_real_mean(f, n)).log.get_or_else(Real.0))
                log_le_sub_one(f(i) / finite_real_mean(f, n),
                    (f(i)).log.get_or_else(Real.0) - (finite_real_mean(f, n)).log.get_or_else(Real.0))
                (f(i)).log.get_or_else(Real.0) - (finite_real_mean(f, n)).log.get_or_else(Real.0) <= f(i) / finite_real_mean(f, n) - Real.1
                l_fn(f, finite_real_mean(f, n))(i) = (f(i)).log.get_or_else(Real.0) - (finite_real_mean(f, n)).log.get_or_else(Real.0)
                r_fn(f, finite_real_mean(f, n))(i) = f(i) / finite_real_mean(f, n) - Real.1
                l_fn(f, finite_real_mean(f, n))(i) <= r_fn(f, finite_real_mean(f, n))(i)
                lte_add_right(l_fn(f, finite_real_mean(f, n))(i), r_fn(f, finite_real_mean(f, n))(i), -(l_fn(f, finite_real_mean(f, n))(i)))
                l_fn(f, finite_real_mean(f, n))(i) + -(l_fn(f, finite_real_mean(f, n))(i)) <= r_fn(f, finite_real_mean(f, n))(i) + -(l_fn(f, finite_real_mean(f, n))(i))
                add_neg_eq_zero(l_fn(f, finite_real_mean(f, n))(i))
                l_fn(f, finite_real_mean(f, n))(i) + -(l_fn(f, finite_real_mean(f, n))(i)) = Real.0
                Real.0 <= r_fn(f, finite_real_mean(f, n))(i) - l_fn(f, finite_real_mean(f, n))(i)
                r_fn(f, finite_real_mean(f, n))(i) - l_fn(f, finite_real_mean(f, n))(i) >= Real.0
                d_fn(f, finite_real_mean(f, n)) = add_fn(r_fn(f, finite_real_mean(f, n)), neg_fn(l_fn(f, finite_real_mean(f, n))))
                add_fn(r_fn(f, finite_real_mean(f, n)), neg_fn(l_fn(f, finite_real_mean(f, n))))(i) =
                    r_fn(f, finite_real_mean(f, n))(i) + neg_fn(l_fn(f, finite_real_mean(f, n)))(i)
                d_fn(f, finite_real_mean(f, n))(i) =
                    r_fn(f, finite_real_mean(f, n))(i) + neg_fn(l_fn(f, finite_real_mean(f, n)))(i)
                neg_fn(l_fn(f, finite_real_mean(f, n)))(i) = -(l_fn(f, finite_real_mean(f, n))(i))
                d_fn(f, finite_real_mean(f, n))(i) =
                    r_fn(f, finite_real_mean(f, n))(i) - l_fn(f, finite_real_mean(f, n))(i)
                d_fn(f, finite_real_mean(f, n))(i) >= Real.0
            }
        }
        forall(i: Nat) { i < n implies d_fn(f, finite_real_mean(f, n))(i) >= Real.0 }
        partial(d_fn(f, finite_real_mean(f, n)), n) = Real.0
        partial_nonneg_eq_zero_imp_each(d_fn(f, finite_real_mean(f, n)), n)
        (forall(i: Nat) { i < n implies d_fn(f, finite_real_mean(f, n))(i) >= Real.0 }) and partial(d_fn(f, finite_real_mean(f, n)), n) = Real.0
        ((forall(i: Nat) { i < n implies d_fn(f, finite_real_mean(f, n))(i) >= Real.0 }) and partial(d_fn(f, finite_real_mean(f, n)), n) = Real.0) implies forall(i: Nat) { i < n implies d_fn(f, finite_real_mean(f, n))(i) = Real.0 }
        forall(i: Nat) { i < n implies d_fn(f, finite_real_mean(f, n))(i) = Real.0 }
        forall(i: Nat) {
            if i < n {
                d_fn(f, finite_real_mean(f, n))(i) = Real.0
                d_fn(f, finite_real_mean(f, n)) = add_fn(r_fn(f, finite_real_mean(f, n)), neg_fn(l_fn(f, finite_real_mean(f, n))))
                add_fn(r_fn(f, finite_real_mean(f, n)), neg_fn(l_fn(f, finite_real_mean(f, n))))(i) =
                    r_fn(f, finite_real_mean(f, n))(i) + neg_fn(l_fn(f, finite_real_mean(f, n)))(i)
                d_fn(f, finite_real_mean(f, n))(i) =
                    r_fn(f, finite_real_mean(f, n))(i) + neg_fn(l_fn(f, finite_real_mean(f, n)))(i)
                neg_fn(l_fn(f, finite_real_mean(f, n)))(i) = -(l_fn(f, finite_real_mean(f, n))(i))
                r_fn(f, finite_real_mean(f, n))(i) - l_fn(f, finite_real_mean(f, n))(i) = Real.0
                sub_eq_zero_imp_eq(r_fn(f, finite_real_mean(f, n))(i), l_fn(f, finite_real_mean(f, n))(i))
                r_fn(f, finite_real_mean(f, n))(i) = l_fn(f, finite_real_mean(f, n))(i)
                l_fn(f, finite_real_mean(f, n))(i) = r_fn(f, finite_real_mean(f, n))(i)
                positive_on(f, n) = forall(j: Nat) { j < n implies f(j) > Real.0 }
                f(i) > Real.0
                div_pos_of_pos_pos(f(i), finite_real_mean(f, n))
                f(i) / finite_real_mean(f, n) > Real.0
                log_div(f(i), finite_real_mean(f, n))
                (f(i) / finite_real_mean(f, n)).log =
                    Option.some((f(i)).log.get_or_else(Real.0) - (finite_real_mean(f, n)).log.get_or_else(Real.0))
                l_fn(f, finite_real_mean(f, n))(i) = (f(i)).log.get_or_else(Real.0) - (finite_real_mean(f, n)).log.get_or_else(Real.0)
                if f(i) / finite_real_mean(f, n) = Real.1 {
                    mul_div_cancel(f(i), finite_real_mean(f, n))
                    finite_real_mean(f, n) * (f(i) / finite_real_mean(f, n)) = f(i)
                    finite_real_mean(f, n) * Real.1 = f(i)
                    mul_one_right(finite_real_mean(f, n))
                    finite_real_mean(f, n) * Real.1 = finite_real_mean(f, n)
                    finite_real_mean(f, n) = f(i)
                    f(i) = finite_real_mean(f, n)
                } else {
                    if not f(i) = finite_real_mean(f, n) {
                        log_lt_sub_one(f(i) / finite_real_mean(f, n),
                            (f(i)).log.get_or_else(Real.0) - (finite_real_mean(f, n)).log.get_or_else(Real.0))
                        (f(i)).log.get_or_else(Real.0) - (finite_real_mean(f, n)).log.get_or_else(Real.0) < f(i) / finite_real_mean(f, n) - Real.1
                        l_fn(f, finite_real_mean(f, n))(i) < r_fn(f, finite_real_mean(f, n))(i)
                        r_fn(f, finite_real_mean(f, n))(i) = l_fn(f, finite_real_mean(f, n))(i)
                        l_fn(f, finite_real_mean(f, n))(i) < l_fn(f, finite_real_mean(f, n))(i)
                        lt_irrefl(l_fn(f, finite_real_mean(f, n))(i))
                        false
                    }
                    f(i) = finite_real_mean(f, n)
                }
            }
        }
    }
}

/// Equality in the AM-GM inequality forces all values to equal the mean.
theorem am_gm_eq_imp_all_eq(f: Nat -> Real, n: Nat) {
    n != Nat.0 and nonnegative_on(f, n)
    and exists(g: Real) {
        (finite_real_product(f, n)).rpow(Real.1 / from_nat[Real](n)) = Option.some(g)
        and g = finite_real_mean(f, n)
    }
    implies forall(i: Nat) { i < n implies f(i) = finite_real_mean(f, n) }
} by {
    if n != Nat.0 and nonnegative_on(f, n)
        and exists(g: Real) {
            (finite_real_product(f, n)).rpow(Real.1 / from_nat[Real](n)) = Option.some(g)
            and g = finite_real_mean(f, n)
        } {
        let g: Real satisfy {
            (finite_real_product(f, n)).rpow(Real.1 / from_nat[Real](n)) = Option.some(g)
            and g = finite_real_mean(f, n)
        }
        if positive_on(f, n) {
            am_gm_eq_imp_all_eq_pos(f, n)
            forall(i: Nat) { i < n implies f(i) = finite_real_mean(f, n) }
        } else {
            not_positive_on_imp_zero_factor(f, n)
            exists(i: Nat) { i < n and f(i) = Real.0 }
            let i: Nat satisfy {
                i < n and f(i) = Real.0
            }
            finite_real_product_zero_of_factor(f, n, i)
            finite_real_product(f, n) = Real.0
            from_nat_real_pos_of_ne_zero(n)
            from_nat[Real](n) > Real.0
            real_one_div_pos(from_nat[Real](n))
            Real.1 / from_nat[Real](n) > Real.0
            rpow_zero_base_pos(Real.1 / from_nat[Real](n))
            (Real.0).rpow(Real.1 / from_nat[Real](n)) = Option.some(Real.0)
            (finite_real_product(f, n)).rpow(Real.1 / from_nat[Real](n)) = Option.some(Real.0)
            Option.some(g) = Option.some(Real.0)
            some_injective[Real](g, Real.0)
            g = Real.0
            g = finite_real_mean(f, n)
            finite_real_mean(f, n) = Real.0
            finite_real_mean_mul_count(f, n)
            finite_real_mean(f, n) * from_nat[Real](n) = partial(f, n)
            mul_zero_left(from_nat[Real](n))
            Real.0 * from_nat[Real](n) = Real.0
            partial(f, n) = Real.0
            nonnegative_on(f, n) = forall(j: Nat) { j < n implies f(j) >= Real.0 }
            forall(j: Nat) { j < n implies f(j) >= Real.0 }
            partial_nonneg_eq_zero_imp_each(f, n)
            (forall(j: Nat) { j < n implies f(j) >= Real.0 }) and partial(f, n) = Real.0
            ((forall(j: Nat) { j < n implies f(j) >= Real.0 }) and partial(f, n) = Real.0) implies forall(j: Nat) { j < n implies f(j) = Real.0 }
            forall(j: Nat) { j < n implies f(j) = Real.0 }
            forall(j: Nat) { j < n implies f(j) = finite_real_mean(f, n) }
        }
    }
}

/// The sequence whose first three values are the arguments.
define tri_fn(a: Real, b: Real, c: Real) -> (Nat -> Real) {
    function(i: Nat) {
        if i = Nat.0 {
            a
        } else {
            if i = Nat.1 {
                b
            } else {
                if i = Nat.2 {
                    c
                } else {
                    Real.0
                }
            }
        }
    }
}

/// The first three values of the three-place sequence are the arguments.
theorem tri_fn_zero(a: Real, b: Real, c: Real) {
    tri_fn(a, b, c)(Nat.0) = a
} by {
    tri_fn(a, b, c)(Nat.0) = a
}

/// The second value of the three-place sequence is the second argument.
theorem tri_fn_one(a: Real, b: Real, c: Real) {
    tri_fn(a, b, c)(Nat.1) = b
} by {
    tri_fn(a, b, c)(Nat.1) = b
}

/// The third value of the three-place sequence is the third argument.
theorem tri_fn_two(a: Real, b: Real, c: Real) {
    tri_fn(a, b, c)(Nat.2) = c
} by {
    if Nat.2 = Nat.0 {
        false
    }
    if Nat.2 = Nat.1 {
        false
    }
    tri_fn(a, b, c)(Nat.2) = c
}

/// The three-variable arithmetic-geometric mean inequality: the geometric mean
/// of three nonnegative reals is at most their arithmetic mean.
theorem am_gm_three(a: Real, b: Real, c: Real) {
    a >= Real.0 and b >= Real.0 and c >= Real.0 implies exists(g: Real) {
        (a * b * c).rpow(Real.1 / from_nat[Real](Nat.3)) = Option.some(g)
        and g <= (a + b + c) / from_nat[Real](Nat.3)
    }
} by {
    if a >= Real.0 and b >= Real.0 and c >= Real.0 {
        forall(i: Nat) {
            if i < Nat.3 {
                lt_suc_right(i, Nat.2)
                i = Nat.2 or i < Nat.2
                if i = Nat.2 {
                    tri_fn(a, b, c)(i) = c
                    c >= Real.0
                    tri_fn(a, b, c)(i) >= Real.0
                } else {
                    i < Nat.2
                    lt_suc_right(i, Nat.1)
                    i = Nat.1 or i < Nat.1
                    if i = Nat.1 {
                        tri_fn(a, b, c)(i) = b
                        b >= Real.0
                        tri_fn(a, b, c)(i) >= Real.0
                    } else {
                        i < Nat.1
                        lt_suc_right(i, Nat.0)
                        i = Nat.0 or i < Nat.0
                        if i = Nat.0 {
                            tri_fn(a, b, c)(i) = a
                            a >= Real.0
                            tri_fn(a, b, c)(i) >= Real.0
                        } else {
                            if not tri_fn(a, b, c)(i) >= Real.0 {
                                i < Nat.0
                                not_lt_zero(i)
                                false
                            }
                            tri_fn(a, b, c)(i) >= Real.0
                        }
                    }
                }
            }
        }
        nonnegative_on(tri_fn(a, b, c), Nat.3)
        nonnegative_on(tri_fn(a, b, c), Nat.3) = forall(i: Nat) {
            i < Nat.3 implies tri_fn(a, b, c)(i) >= Real.0
        }
        Nat.3 != Nat.0
        am_gm(tri_fn(a, b, c), Nat.3)
        exists(g: Real) {
            (finite_real_product(tri_fn(a, b, c), Nat.3)).rpow(Real.1 / from_nat[Real](Nat.3)) = Option.some(g)
            and g <= finite_real_mean(tri_fn(a, b, c), Nat.3)
        }
        let g: Real satisfy {
            (finite_real_product(tri_fn(a, b, c), Nat.3)).rpow(Real.1 / from_nat[Real](Nat.3)) = Option.some(g)
            and g <= finite_real_mean(tri_fn(a, b, c), Nat.3)
        }
        finite_real_product_zero(tri_fn(a, b, c))
        finite_real_product(tri_fn(a, b, c), Nat.0) = Real.1
        tri_fn(a, b, c)(Nat.0) = a
        finite_real_product_suc(tri_fn(a, b, c), Nat.0)
        finite_real_product(tri_fn(a, b, c), Nat.1) = finite_real_product(tri_fn(a, b, c), Nat.0) * tri_fn(a, b, c)(Nat.0)
        finite_real_product(tri_fn(a, b, c), Nat.1) = Real.1 * a
        mul_one_left(a)
        Real.1 * a = a
        finite_real_product(tri_fn(a, b, c), Nat.1) = a
        tri_fn(a, b, c)(Nat.1) = b
        finite_real_product_suc(tri_fn(a, b, c), Nat.1)
        finite_real_product(tri_fn(a, b, c), Nat.2) = finite_real_product(tri_fn(a, b, c), Nat.1) * tri_fn(a, b, c)(Nat.1)
        finite_real_product(tri_fn(a, b, c), Nat.2) = a * b
        tri_fn(a, b, c)(Nat.2) = c
        finite_real_product_suc(tri_fn(a, b, c), Nat.2)
        finite_real_product(tri_fn(a, b, c), Nat.3) = finite_real_product(tri_fn(a, b, c), Nat.2) * tri_fn(a, b, c)(Nat.2)
        finite_real_product(tri_fn(a, b, c), Nat.3) = (a * b) * c
        (a * b) * c = a * b * c
        finite_real_product(tri_fn(a, b, c), Nat.3) = a * b * c
        (finite_real_product(tri_fn(a, b, c), Nat.3)).rpow(Real.1 / from_nat[Real](Nat.3)) = Option.some(g)
        (a * b * c).rpow(Real.1 / from_nat[Real](Nat.3)) = Option.some(g)
        partial_zero(tri_fn(a, b, c))
        partial(tri_fn(a, b, c), Nat.0) = Real.0
        partial_split_last(tri_fn(a, b, c), Nat.0)
        partial(tri_fn(a, b, c), Nat.1) = partial(tri_fn(a, b, c), Nat.0) + tri_fn(a, b, c)(Nat.0)
        partial(tri_fn(a, b, c), Nat.1) = Real.0 + a
        Real.0 + a = a
        partial(tri_fn(a, b, c), Nat.1) = a
        partial_split_last(tri_fn(a, b, c), Nat.1)
        partial(tri_fn(a, b, c), Nat.2) = partial(tri_fn(a, b, c), Nat.1) + tri_fn(a, b, c)(Nat.1)
        partial(tri_fn(a, b, c), Nat.2) = a + b
        partial_split_last(tri_fn(a, b, c), Nat.2)
        partial(tri_fn(a, b, c), Nat.3) = partial(tri_fn(a, b, c), Nat.2) + tri_fn(a, b, c)(Nat.2)
        partial(tri_fn(a, b, c), Nat.3) = (a + b) + c
        (a + b) + c = a + b + c
        partial(tri_fn(a, b, c), Nat.3) = a + b + c
        finite_real_mean(tri_fn(a, b, c), Nat.3) = partial(tri_fn(a, b, c), Nat.3) / from_nat[Real](Nat.3)
        finite_real_mean(tri_fn(a, b, c), Nat.3) = (a + b + c) / from_nat[Real](Nat.3)
        g <= finite_real_mean(tri_fn(a, b, c), Nat.3)
        g <= (a + b + c) / from_nat[Real](Nat.3)
        exists(h: Real) {
            (a * b * c).rpow(Real.1 / from_nat[Real](Nat.3)) = Option.some(h)
            and h <= (a + b + c) / from_nat[Real](Nat.3)
        }
    }
}
