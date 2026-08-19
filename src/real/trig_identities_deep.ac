from nat import Nat, from_nat, from_nat_add, from_nat_zero, from_nat_one
from real.real_field import Real, div_mul_cancel_left, mul_div_cancel, div_cancel_common, zero_is_different_than_one, mul_div, mul_inverse
from real.real_ring import real_mul_comm, mul_distrib_left, mul_one_left, mul_zero_left, mul_neg_one_left, mul_neg_right, mul_neg_left, mul_sub_distrib_left, mul_zero_right
from real.real_base import add_comm, add_assoc, add_zero_left, add_zero_right, add_neg_eq_zero, neg_distrib, neg_zero, neg_neg, sub_cancels
from algebra.add_comm_group import sub_add_sub, sub_add_cancel
from algebra.field.field import mul_not_zero, unique_inverse
from real.trig import sin_neg, cos_neg, sin_zero, cos_zero
from real.trig_identities import sin_add, cos_add, sin_sq_add_cos_sq, div_add_same_denom, div_sub_same_denom, signed_mul_combine
from real.exp import two, two_nonzero, pow_suc, three
from real.pi import pi, pi_over_two, sin_pi_zero, cos_pi_neg_one, cos_pi_over_two_zero, sin_pi_over_two_one, sin_two_pi_zero, cos_two_pi_one
from real.trig_period import sin_add_two_pi, cos_add_two_pi, sin_pi_over_two_sub, sin_add_pi_over_two, cos_add_pi_over_two, sin_add_pi, cos_add_pi
from real.derivative_basic import has_derivative_at
from real.derivative_trig import sin_has_derivative_at, cos_has_derivative_at
from real.derivative_affine_named import affine_real, affine_real_has_derivative_at, derivative_affine_real_after
from real.derivative_chain import derivative_compose
from real.calculus_quotient_api import quotient_has_derivative_at
from real.derivative_quotient import pointwise_div_real, pointwise_div_real_apply
from data.basic.functions import compose, function_extensionality, function_eq_transport_predicate_rev

numerals Real

/// Four as a real number.
let four = Real.1 + Real.1 + Real.1 + Real.1

// Small algebraic helpers for rearranging sums and differences of real
// numbers.  These are stated generically and reused by the trig identities
// below.

/// Negating a difference swaps its terms: -(a - b) = b - a.
theorem neg_diff(a: Real, b: Real) {
    -(a - b) = b - a
} by {
    a - b = a + -b
    -(a - b) = -(a + -b)
    neg_distrib(a, -b)
    -(a + -b) = -a + -(-b)
    neg_neg(b)
    -(-b) = b
    -a + -(-b) = -a + b
    -(a - b) = -a + b
    add_comm(-a, b)
    -a + b = b + -a
    b - a = b + -a
    -(a - b) = b - a
}

/// Subtracting a common minuend cancels: (a - b) - (a - c) = c - b.
theorem sub_sub_cancel(a: Real, b: Real, c: Real) {
    (a - b) - (a - c) = c - b
} by {
    neg_distrib(a, -c)
    -(a + -c) = -a + -(-c)
    neg_neg(c)
    -(-c) = c
    -a + -(-c) = -a + c
    a - c = a + -c
    -(a - c) = -(a + -c)
    (a - b) - (a - c) = (a + -b) + -(a - c)
    (a + -b) + -(a - c) = (a + -b) + (-a + c)
    add_assoc(a, -b, -a + c)
    (a + -b) + (-a + c) = a + (-b + (-a + c))
    add_assoc(-b, -a, c)
    -b + (-a + c) = (-b + -a) + c
    add_comm(-b, -a)
    -b + -a = -a + -b
    (-b + -a) + c = (-a + -b) + c
    a + (-b + (-a + c)) = a + ((-a + -b) + c)
    add_assoc(a, -a + -b, c)
    a + ((-a + -b) + c) = (a + (-a + -b)) + c
    add_assoc(a, -a, -b)
    (a + -a) + -b = a + (-a + -b)
    add_neg_eq_zero(a)
    a + -a = Real.0
    ((a + -a) + -b) + c = (Real.0 + -b) + c
    add_zero_left(-b)
    Real.0 + -b = -b
    (Real.0 + -b) + c = -b + c
    c - b = c + -b
    (a - b) - (a - c) = c - b
}

/// A difference cancels when subtracted from its minuend: a - (a - b) = b.
theorem sub_minus_sub(a: Real, b: Real) {
    a - (a - b) = b
} by {
    sub_sub_cancel(a, Real.0, b)
    (a - Real.0) - (a - b) = b - Real.0
    neg_zero
    -Real.0 = Real.0
    a - Real.0 = a + -Real.0
    a + -Real.0 = a + Real.0
    add_zero_right(a)
    a + Real.0 = a
    a - Real.0 = a
    b - Real.0 = b + -Real.0
    b + -Real.0 = b + Real.0
    add_zero_right(b)
    b + Real.0 = b
    b - Real.0 = b
    a - (a - b) = b
}

/// A sum and its mirrored difference add to twice the first summand:
/// (a + b) + (a - b) = 2a.
theorem add_sum_diff(a: Real, b: Real) {
    (a + b) + (a - b) = two * a
} by {
    sub_add_cancel(a, b)
    (a - b) + b = a
    add_comm(b, a - b)
    b + (a - b) = (a - b) + b
    b + (a - b) = a
    add_assoc(a, b, a - b)
    (a + b) + (a - b) = a + (b + (a - b))
    (a + b) + (a - b) = a + a
    a + a = two * a
    (a + b) + (a - b) = two * a
}

/// A sum minus its mirrored difference is twice the second summand:
/// (a + b) - (a - b) = 2b.
theorem sub_sum_diff(a: Real, b: Real) {
    (a + b) - (a - b) = two * b
} by {
    neg_neg(b)
    -(-b) = b
    a - (-b) = a + b
    sub_sub_cancel(a, -b, b)
    (a - (-b)) - (a - b) = b - (-b)
    (a + b) - (a - b) = (a - (-b)) - (a - b)
    b - (-b) = b + b
    b + b = two * b
    (a + b) - (a - b) = two * b
}

/// The sum of a difference and its negated-difference mirror is twice the
/// common minuend: (a - b) + (a - (-b)) = 2a.
theorem add_diff_neg_diff(a: Real, b: Real) {
    (a - b) + (a - (-b)) = two * a
} by {
    sub_add_sub(a, b, a, -b)
    (a - b) + (a - (-b)) = (a + a) - (b + -b)
    add_neg_eq_zero(b)
    b + -b = Real.0
    (a + a) - (b + -b) = (a + a) - Real.0
    (a + a) - Real.0 = a + a
    a + a = two * a
    (a - b) + (a - (-b)) = two * a
}

/// The difference of a negated-difference mirror and the difference itself is
/// twice the common subtrahend: (a - (-b)) - (a - b) = 2b.
theorem sub_neg_diff_sub(a: Real, b: Real) {
    (a - (-b)) - (a - b) = two * b
} by {
    sub_sub_cancel(a, -b, b)
    (a - (-b)) - (a - b) = b - (-b)
    neg_neg(b)
    -(-b) = b
    b - (-b) = b + b
    b + b = two * b
    (a - (-b)) - (a - b) = two * b
}

/// The product of two negated numbers is the product of the originals:
/// (-a)(-b) = ab.
theorem neg_mul_neg(a: Real, b: Real) {
    (-a) * (-b) = a * b
} by {
    mul_neg_one_left(a)
    -Real.1 * a = -a
    mul_neg_one_left(b)
    -Real.1 * b = -b
    (-a) * (-b) = (-Real.1 * a) * (-Real.1 * b)
    signed_mul_combine(-Real.1, -Real.1, a, b)
    (-Real.1 * a) * (-Real.1 * b) = (-Real.1 * -Real.1) * (a * b)
    mul_neg_one_left(-Real.1)
    -Real.1 * -Real.1 = --Real.1
    neg_neg(Real.1)
    --Real.1 = Real.1
    (-Real.1 * -Real.1) * (a * b) = Real.1 * (a * b)
    Real.1 * (a * b) = a * b
    (-a) * (-b) = a * b
}

/// One and two times a summand combine to three times it: a + 2a = 3a.
theorem one_add_two_mul(a: Real) {
    a + two * a = three * a
}

/// One and three times a summand combine to four times it: a + 3a = 4a.
theorem one_add_three_mul(a: Real) {
    a + three * a = four * a
}

/// Three and one times a summand combine to four times it: 3a + a = 4a.
theorem three_add_one_mul(a: Real) {
    three * a + a = four * a
} by {
    one_add_three_mul(a)
    a + three * a = four * a
    add_comm(a, three * a)
    a + three * a = three * a + a
    three * a + a = four * a
}

/// Subtracting a difference expands into a sum: a - (b - c) = a - b + c.
theorem sub_diff_add(a: Real, b: Real, c: Real) {
    a - (b - c) = a - b + c
} by {
    b - c = b + -c
    a - (b - c) = a - (b + -c)
    a - (b + -c) = a + -(b + -c)
    neg_distrib(b, -c)
    -(b + -c) = -b + -(-c)
    neg_neg(c)
    -(-c) = c
    -b + -(-c) = -b + c
    a + -(b + -c) = a + (-b + c)
    a + (-b + c) = a + -b + c
    a - b + c = a + -b + c
    a - (b - c) = a - b + c
}

// Double-angle formulas, derived directly from the addition formulas with
// y = x.

/// The sine of twice an angle is twice the product of sine and cosine.
theorem sin_two(x: Real) {
    (two * x).sin = two * x.sin * x.cos
} by {
    sin_add(x, x)
    (x + x).sin = x.sin * x.cos + x.cos * x.sin
    two = Real.1 + Real.1
    two * x = x + x
    (two * x).sin = x.sin * x.cos + x.cos * x.sin
    x.cos * x.sin = x.sin * x.cos
    x.sin * x.cos + x.cos * x.sin = x.sin * x.cos + x.sin * x.cos
    x.sin * x.cos + x.sin * x.cos = two * (x.sin * x.cos)
    two * (x.sin * x.cos) = two * x.sin * x.cos
    (two * x).sin = two * x.sin * x.cos
}

/// The cosine of twice an angle is the difference of the squares.
theorem cos_two(x: Real) {
    (two * x).cos = x.cos.pow(Nat.2) - x.sin.pow(Nat.2)
} by {
    cos_add(x, x)
    (x + x).cos = x.cos * x.cos - x.sin * x.sin
    two = Real.1 + Real.1
    two * x = x + x
    (two * x).cos = x.cos * x.cos - x.sin * x.sin
    pow_suc(x.cos, Nat.1)
    x.cos.pow(Nat.2) = x.cos * x.cos
    pow_suc(x.sin, Nat.1)
    x.sin.pow(Nat.2) = x.sin * x.sin
    (two * x).cos = x.cos.pow(Nat.2) - x.sin.pow(Nat.2)
}

/// The cosine of twice an angle in terms of cosine squared alone.
theorem cos_two_alt1(x: Real) {
    (two * x).cos = two * x.cos.pow(Nat.2) - Real.1
} by {
    cos_two(x)
    (two * x).cos = x.cos.pow(Nat.2) - x.sin.pow(Nat.2)
    sin_sq_add_cos_sq(x)
    x.sin.pow(Nat.2) + x.cos.pow(Nat.2) = Real.1
    x.sin.pow(Nat.2) = Real.1 - x.cos.pow(Nat.2)
    x.cos.pow(Nat.2) - x.sin.pow(Nat.2) = x.cos.pow(Nat.2) - (Real.1 - x.cos.pow(Nat.2))
    neg_diff(Real.1, x.cos.pow(Nat.2))
    -(Real.1 - x.cos.pow(Nat.2)) = x.cos.pow(Nat.2) - Real.1
    x.cos.pow(Nat.2) - (Real.1 - x.cos.pow(Nat.2)) = x.cos.pow(Nat.2) + (x.cos.pow(Nat.2) - Real.1)
    x.cos.pow(Nat.2) + (x.cos.pow(Nat.2) - Real.1) = x.cos.pow(Nat.2) + x.cos.pow(Nat.2) - Real.1
    x.cos.pow(Nat.2) + x.cos.pow(Nat.2) = two * x.cos.pow(Nat.2)
    x.cos.pow(Nat.2) - (Real.1 - x.cos.pow(Nat.2)) = two * x.cos.pow(Nat.2) - Real.1
    x.cos.pow(Nat.2) - x.sin.pow(Nat.2) = two * x.cos.pow(Nat.2) - Real.1
    (two * x).cos = two * x.cos.pow(Nat.2) - Real.1
}

/// The cosine of twice an angle in terms of sine squared alone.
theorem cos_two_alt2(x: Real) {
    (two * x).cos = Real.1 - two * x.sin.pow(Nat.2)
} by {
    cos_two(x)
    (two * x).cos = x.cos.pow(Nat.2) - x.sin.pow(Nat.2)
    sin_sq_add_cos_sq(x)
    x.sin.pow(Nat.2) + x.cos.pow(Nat.2) = Real.1
    x.cos.pow(Nat.2) = Real.1 - x.sin.pow(Nat.2)
    x.cos.pow(Nat.2) - x.sin.pow(Nat.2) = (Real.1 - x.sin.pow(Nat.2)) - x.sin.pow(Nat.2)
    (Real.1 - x.sin.pow(Nat.2)) - x.sin.pow(Nat.2) = Real.1 - (x.sin.pow(Nat.2) + x.sin.pow(Nat.2))
    x.sin.pow(Nat.2) + x.sin.pow(Nat.2) = two * x.sin.pow(Nat.2)
    Real.1 - (x.sin.pow(Nat.2) + x.sin.pow(Nat.2)) = Real.1 - two * x.sin.pow(Nat.2)
    (Real.1 - x.sin.pow(Nat.2)) - x.sin.pow(Nat.2) = Real.1 - two * x.sin.pow(Nat.2)
    x.cos.pow(Nat.2) - x.sin.pow(Nat.2) = Real.1 - two * x.sin.pow(Nat.2)
    (two * x).cos = Real.1 - two * x.sin.pow(Nat.2)
}

// Half-angle (power-reduction) formulas, rearranged from the double-angle
// formulas.

/// The square of cosine is one plus cosine of twice the angle, over two.
theorem cos_sq_half(x: Real) {
    x.cos.pow(Nat.2) = (Real.1 + (two * x).cos) / two
} by {
    cos_two_alt1(x)
    (two * x).cos = two * x.cos.pow(Nat.2) - Real.1
    Real.1 + (two * x).cos = Real.1 + (two * x.cos.pow(Nat.2) - Real.1)
    Real.1 + (two * x.cos.pow(Nat.2) - Real.1) = Real.1 + two * x.cos.pow(Nat.2) - Real.1
    Real.1 + two * x.cos.pow(Nat.2) - Real.1 = two * x.cos.pow(Nat.2)
    Real.1 + (two * x.cos.pow(Nat.2) - Real.1) = two * x.cos.pow(Nat.2)
    Real.1 + (two * x).cos = two * x.cos.pow(Nat.2)
    two_nonzero
    two != Real.0
    div_mul_cancel_left(two, x.cos.pow(Nat.2))
    (two * x.cos.pow(Nat.2)) / two = x.cos.pow(Nat.2)
    (Real.1 + (two * x).cos) / two = (two * x.cos.pow(Nat.2)) / two
    (Real.1 + (two * x).cos) / two = x.cos.pow(Nat.2)
}

/// The square of sine is one minus cosine of twice the angle, over two.
theorem sin_sq_half(x: Real) {
    x.sin.pow(Nat.2) = (Real.1 - (two * x).cos) / two
} by {
    cos_two_alt2(x)
    (two * x).cos = Real.1 - two * x.sin.pow(Nat.2)
    Real.1 - (two * x).cos = Real.1 - (Real.1 - two * x.sin.pow(Nat.2))
    sub_minus_sub(Real.1, two * x.sin.pow(Nat.2))
    Real.1 - (Real.1 - two * x.sin.pow(Nat.2)) = two * x.sin.pow(Nat.2)
    Real.1 - (two * x).cos = two * x.sin.pow(Nat.2)
    two_nonzero
    two != Real.0
    div_mul_cancel_left(two, x.sin.pow(Nat.2))
    (two * x.sin.pow(Nat.2)) / two = x.sin.pow(Nat.2)
    (Real.1 - (two * x).cos) / two = (two * x.sin.pow(Nat.2)) / two
    (Real.1 - (two * x).cos) / two = x.sin.pow(Nat.2)
}

// Product-to-sum formulas, from adding and subtracting the addition formulas.

/// Sine times cosine is half the sum of the sine of the sum and the sine of
/// the difference.
theorem sin_mul_cos(x: Real, y: Real) {
    x.sin * y.cos = ((x + y).sin + (x - y).sin) / two
} by {
    sin_add(x, y)
    (x + y).sin = x.sin * y.cos + x.cos * y.sin
    sin_add(x, -y)
    (x + (-y)).sin = x.sin * (-y).cos + x.cos * (-y).sin
    x + (-y) = x - y
    (x - y).sin = x.sin * (-y).cos + x.cos * (-y).sin
    cos_neg(y)
    (-y).cos = y.cos
    sin_neg(y)
    (-y).sin = -y.sin
    (x - y).sin = x.sin * y.cos + x.cos * (-y.sin)
    mul_neg_right(x.cos, y.sin)
    x.cos * (-y.sin) = -(x.cos * y.sin)
    (x - y).sin = x.sin * y.cos - x.cos * y.sin
    (x + y).sin + (x - y).sin = (x.sin * y.cos + x.cos * y.sin) + (x - y).sin
    (x + y).sin + (x - y).sin =
        (x.sin * y.cos + x.cos * y.sin) + (x.sin * y.cos - x.cos * y.sin)
    add_sum_diff(x.sin * y.cos, x.cos * y.sin)
    (x.sin * y.cos + x.cos * y.sin) + (x.sin * y.cos - x.cos * y.sin) =
        two * (x.sin * y.cos)
    (x + y).sin + (x - y).sin = two * (x.sin * y.cos)
    two_nonzero
    two != Real.0
    div_mul_cancel_left(two, x.sin * y.cos)
    (two * (x.sin * y.cos)) / two = x.sin * y.cos
    ((x + y).sin + (x - y).sin) / two = (two * (x.sin * y.cos)) / two
    ((x + y).sin + (x - y).sin) / two = x.sin * y.cos
}

/// Cosine times sine is half the difference of the sine of the sum and the
/// sine of the difference.
theorem cos_mul_sin(x: Real, y: Real) {
    x.cos * y.sin = ((x + y).sin - (x - y).sin) / two
} by {
    sin_add(x, y)
    (x + y).sin = x.sin * y.cos + x.cos * y.sin
    sin_add(x, -y)
    (x + (-y)).sin = x.sin * (-y).cos + x.cos * (-y).sin
    x + (-y) = x - y
    (x - y).sin = x.sin * (-y).cos + x.cos * (-y).sin
    cos_neg(y)
    (-y).cos = y.cos
    sin_neg(y)
    (-y).sin = -y.sin
    (x - y).sin = x.sin * y.cos + x.cos * (-y.sin)
    mul_neg_right(x.cos, y.sin)
    x.cos * (-y.sin) = -(x.cos * y.sin)
    (x - y).sin = x.sin * y.cos - x.cos * y.sin
    (x + y).sin - (x - y).sin = (x.sin * y.cos + x.cos * y.sin) - (x - y).sin
    (x + y).sin - (x - y).sin =
        (x.sin * y.cos + x.cos * y.sin) - (x.sin * y.cos - x.cos * y.sin)
    sub_sum_diff(x.sin * y.cos, x.cos * y.sin)
    (x.sin * y.cos + x.cos * y.sin) - (x.sin * y.cos - x.cos * y.sin) =
        two * (x.cos * y.sin)
    (x + y).sin - (x - y).sin = two * (x.cos * y.sin)
    two_nonzero
    two != Real.0
    div_mul_cancel_left(two, x.cos * y.sin)
    (two * (x.cos * y.sin)) / two = x.cos * y.sin
    ((x + y).sin - (x - y).sin) / two = (two * (x.cos * y.sin)) / two
    ((x + y).sin - (x - y).sin) / two = x.cos * y.sin
}

/// Cosine times cosine is half the sum of the cosines of the sum and the
/// difference.
theorem cos_mul_cos(x: Real, y: Real) {
    x.cos * y.cos = ((x + y).cos + (x - y).cos) / two
} by {
    cos_add(x, y)
    (x + y).cos = x.cos * y.cos - x.sin * y.sin
    cos_add(x, -y)
    (x + (-y)).cos = x.cos * (-y).cos - x.sin * (-y).sin
    x + (-y) = x - y
    (x - y).cos = x.cos * (-y).cos - x.sin * (-y).sin
    cos_neg(y)
    (-y).cos = y.cos
    sin_neg(y)
    (-y).sin = -y.sin
    (x - y).cos = x.cos * y.cos - x.sin * (-y.sin)
    (x + y).cos + (x - y).cos = (x.cos * y.cos - x.sin * y.sin) + (x - y).cos
    (x + y).cos + (x - y).cos =
        (x.cos * y.cos - x.sin * y.sin) + (x.cos * y.cos - x.sin * (-y.sin))
    mul_neg_right(x.sin, y.sin)
    x.sin * (-y.sin) = -(x.sin * y.sin)
    (x.cos * y.cos - x.sin * y.sin) + (x.cos * y.cos - x.sin * (-y.sin)) =
        (x.cos * y.cos - x.sin * y.sin) + (x.cos * y.cos - (-(x.sin * y.sin)))
    add_diff_neg_diff(x.cos * y.cos, x.sin * y.sin)
    (x.cos * y.cos - x.sin * y.sin) + (x.cos * y.cos - (-(x.sin * y.sin))) =
        two * (x.cos * y.cos)
    (x.cos * y.cos - x.sin * y.sin) + (x.cos * y.cos - x.sin * (-y.sin)) =
        two * (x.cos * y.cos)
    (x + y).cos + (x - y).cos = two * (x.cos * y.cos)
    two_nonzero
    two != Real.0
    div_mul_cancel_left(two, x.cos * y.cos)
    (two * (x.cos * y.cos)) / two = x.cos * y.cos
    ((x + y).cos + (x - y).cos) / two = (two * (x.cos * y.cos)) / two
    ((x + y).cos + (x - y).cos) / two = x.cos * y.cos
}

/// Sine times sine is half the difference of the cosines of the difference and
/// the sum.
theorem sin_mul_sin(x: Real, y: Real) {
    x.sin * y.sin = ((x - y).cos - (x + y).cos) / two
} by {
    cos_add(x, -y)
    (x + (-y)).cos = x.cos * (-y).cos - x.sin * (-y).sin
    x + (-y) = x - y
    (x - y).cos = x.cos * (-y).cos - x.sin * (-y).sin
    cos_neg(y)
    (-y).cos = y.cos
    sin_neg(y)
    (-y).sin = -y.sin
    (x - y).cos = x.cos * y.cos - x.sin * (-y.sin)
    cos_add(x, y)
    (x + y).cos = x.cos * y.cos - x.sin * y.sin
    (x - y).cos - (x + y).cos = (x - y).cos - (x.cos * y.cos - x.sin * y.sin)
    (x - y).cos - (x + y).cos =
        (x.cos * y.cos - x.sin * (-y.sin)) - (x.cos * y.cos - x.sin * y.sin)
    mul_neg_right(x.sin, y.sin)
    x.sin * (-y.sin) = -(x.sin * y.sin)
    (x.cos * y.cos - x.sin * (-y.sin)) - (x.cos * y.cos - x.sin * y.sin) =
        (x.cos * y.cos - (-(x.sin * y.sin))) - (x.cos * y.cos - x.sin * y.sin)
    sub_neg_diff_sub(x.cos * y.cos, x.sin * y.sin)
    (x.cos * y.cos - (-(x.sin * y.sin))) - (x.cos * y.cos - x.sin * y.sin) =
        two * (x.sin * y.sin)
    (x.cos * y.cos - x.sin * (-y.sin)) - (x.cos * y.cos - x.sin * y.sin) =
        two * (x.sin * y.sin)
    (x - y).cos - (x + y).cos = two * (x.sin * y.sin)
    two_nonzero
    two != Real.0
    div_mul_cancel_left(two, x.sin * y.sin)
    (two * (x.sin * y.sin)) / two = x.sin * y.sin
    ((x - y).cos - (x + y).cos) / two = (two * (x.sin * y.sin)) / two
    ((x - y).cos - (x + y).cos) / two = x.sin * y.sin
}

// Further consequences.

/// The sine of x plus itself is the sine of twice x.
theorem sin_add_self(x: Real) {
    (x + x).sin = (two * x).sin
} by {
    two = Real.1 + Real.1
    two * x = x + x
    (two * x).sin = (x + x).sin
    (x + x).sin = (two * x).sin
}

/// Two pi is a period of sine: (x + 2pi).sin = x.sin.
theorem sin_add_two_pi_two(x: Real) {
    (x + two * pi).sin = x.sin
} by {
    sin_add_two_pi(x)
    (x + (pi + pi)).sin = x.sin
    two = Real.1 + Real.1
    two * pi = pi + pi
    (x + two * pi).sin = x.sin
}

/// The average plus the half-difference is the larger term:
/// (x + y)/2 + (x - y)/2 = x.
theorem half_sum_add(x: Real, y: Real) {
    (x + y) / two + (x - y) / two = x
} by {
    two_nonzero
    two != Real.0
    div_add_same_denom(Real.1, x + y, x - y, two)
    Real.1 * (x + y) / two + Real.1 * (x - y) / two = Real.1 * ((x + y) + (x - y)) / two
    Real.1 * (x + y) / two + Real.1 * (x - y) / two = (x + y) / two + (x - y) / two
    Real.1 * ((x + y) + (x - y)) / two = ((x + y) + (x - y)) / two
    (x + y) / two + (x - y) / two = ((x + y) + (x - y)) / two
    add_sum_diff(x, y)
    (x + y) + (x - y) = two * x
    ((x + y) + (x - y)) / two = (two * x) / two
    div_mul_cancel_left(two, x)
    (two * x) / two = x
    ((x + y) + (x - y)) / two = x
    (x + y) / two + (x - y) / two = x
}

/// The average minus the half-difference is the smaller term:
/// (x + y)/2 - (x - y)/2 = y.
theorem half_sum_sub(x: Real, y: Real) {
    (x + y) / two - (x - y) / two = y
} by {
    two_nonzero
    two != Real.0
    div_sub_same_denom(Real.1, Real.1, x + y, x - y, two)
    Real.1 * (x + y) / two - Real.1 * (x - y) / two = (Real.1 * (x + y) - Real.1 * (x - y)) / two
    Real.1 * (x + y) / two - Real.1 * (x - y) / two = (x + y) / two - (x - y) / two
    (Real.1 * (x + y) - Real.1 * (x - y)) / two = ((x + y) - (x - y)) / two
    (x + y) / two - (x - y) / two = ((x + y) - (x - y)) / two
    sub_sum_diff(x, y)
    (x + y) - (x - y) = two * y
    ((x + y) - (x - y)) / two = (two * y) / two
    div_mul_cancel_left(two, y)
    (two * y) / two = y
    ((x + y) - (x - y)) / two = y
    (x + y) / two - (x - y) / two = y
}

/// The sum of two sines is twice the sine of the average times the cosine of
/// the half-difference.
theorem sin_add_sin(x: Real, y: Real) {
    x.sin + y.sin = two * ((x + y) / two).sin * ((x - y) / two).cos
} by {
    sin_mul_cos((x + y) / two, (x - y) / two)
    ((x + y) / two).sin * ((x - y) / two).cos =
        (((x + y) / two + (x - y) / two).sin + ((x + y) / two - (x - y) / two).sin) / two
    half_sum_add(x, y)
    (x + y) / two + (x - y) / two = x
    half_sum_sub(x, y)
    (x + y) / two - (x - y) / two = y
    ((x + y) / two + (x - y) / two).sin = x.sin
    ((x + y) / two - (x - y) / two).sin = y.sin
    ((x + y) / two).sin * ((x - y) / two).cos = (x.sin + y.sin) / two
    mul_div_cancel(x.sin + y.sin, two)
    two * ((x.sin + y.sin) / two) = x.sin + y.sin
    two * ((x.sin + y.sin) / two) = two * (((x + y) / two).sin * ((x - y) / two).cos)
    two * (((x + y) / two).sin * ((x - y) / two).cos) = x.sin + y.sin
    two * ((x + y) / two).sin * ((x - y) / two).cos = two * (((x + y) / two).sin * ((x - y) / two).cos)
    two * ((x + y) / two).sin * ((x - y) / two).cos = x.sin + y.sin
}

/// The difference of two sines is twice the cosine of the average times the
/// sine of the half-difference.
theorem sin_sub_sin(x: Real, y: Real) {
    x.sin - y.sin = two * ((x + y) / two).cos * ((x - y) / two).sin
} by {
    cos_mul_sin((x + y) / two, (x - y) / two)
    ((x + y) / two).cos * ((x - y) / two).sin =
        (((x + y) / two + (x - y) / two).sin - ((x + y) / two - (x - y) / two).sin) / two
    half_sum_add(x, y)
    (x + y) / two + (x - y) / two = x
    half_sum_sub(x, y)
    (x + y) / two - (x - y) / two = y
    ((x + y) / two + (x - y) / two).sin = x.sin
    ((x + y) / two - (x - y) / two).sin = y.sin
    ((x + y) / two).cos * ((x - y) / two).sin = (x.sin - y.sin) / two
    mul_div_cancel(x.sin - y.sin, two)
    two * ((x.sin - y.sin) / two) = x.sin - y.sin
    two * ((x.sin - y.sin) / two) = two * (((x + y) / two).cos * ((x - y) / two).sin)
    two * (((x + y) / two).cos * ((x - y) / two).sin) = x.sin - y.sin
    two * ((x + y) / two).cos * ((x - y) / two).sin = two * (((x + y) / two).cos * ((x - y) / two).sin)
    two * ((x + y) / two).cos * ((x - y) / two).sin = x.sin - y.sin
}

/// The sum of two cosines is twice the cosine of the average times the cosine
/// of the half-difference.
theorem cos_add_cos(x: Real, y: Real) {
    x.cos + y.cos = two * ((x + y) / two).cos * ((x - y) / two).cos
} by {
    cos_mul_cos((x + y) / two, (x - y) / two)
    ((x + y) / two).cos * ((x - y) / two).cos =
        (((x + y) / two + (x - y) / two).cos + ((x + y) / two - (x - y) / two).cos) / two
    half_sum_add(x, y)
    (x + y) / two + (x - y) / two = x
    half_sum_sub(x, y)
    (x + y) / two - (x - y) / two = y
    ((x + y) / two + (x - y) / two).cos = x.cos
    ((x + y) / two - (x - y) / two).cos = y.cos
    ((x + y) / two).cos * ((x - y) / two).cos = (x.cos + y.cos) / two
    mul_div_cancel(x.cos + y.cos, two)
    two * ((x.cos + y.cos) / two) = x.cos + y.cos
    two * ((x.cos + y.cos) / two) = two * (((x + y) / two).cos * ((x - y) / two).cos)
    two * (((x + y) / two).cos * ((x - y) / two).cos) = x.cos + y.cos
    two * ((x + y) / two).cos * ((x - y) / two).cos = two * (((x + y) / two).cos * ((x - y) / two).cos)
    two * ((x + y) / two).cos * ((x - y) / two).cos = x.cos + y.cos
}

/// The difference of two cosines is minus twice the sine of the average times
/// the sine of the half-difference.
theorem cos_sub_cos(x: Real, y: Real) {
    x.cos - y.cos = -(two * ((x + y) / two).sin * ((x - y) / two).sin)
} by {
    sin_mul_sin((x + y) / two, (x - y) / two)
    ((x + y) / two).sin * ((x - y) / two).sin =
        (((x + y) / two - (x - y) / two).cos - ((x + y) / two + (x - y) / two).cos) / two
    half_sum_add(x, y)
    (x + y) / two + (x - y) / two = x
    half_sum_sub(x, y)
    (x + y) / two - (x - y) / two = y
    ((x + y) / two - (x - y) / two).cos = y.cos
    ((x + y) / two + (x - y) / two).cos = x.cos
    ((x + y) / two).sin * ((x - y) / two).sin = (y.cos - x.cos) / two
    mul_div_cancel(y.cos - x.cos, two)
    two * ((y.cos - x.cos) / two) = y.cos - x.cos
    two * ((y.cos - x.cos) / two) = two * (((x + y) / two).sin * ((x - y) / two).sin)
    two * (((x + y) / two).sin * ((x - y) / two).sin) = y.cos - x.cos
    neg_diff(y.cos, x.cos)
    -(y.cos - x.cos) = x.cos - y.cos
    -(two * (((x + y) / two).sin * ((x - y) / two).sin)) = -(y.cos - x.cos)
    -(two * (((x + y) / two).sin * ((x - y) / two).sin)) = x.cos - y.cos
    two * ((x + y) / two).sin * ((x - y) / two).sin = two * (((x + y) / two).sin * ((x - y) / two).sin)
    -(two * ((x + y) / two).sin * ((x - y) / two).sin) = -(two * (((x + y) / two).sin * ((x - y) / two).sin))
    -(two * ((x + y) / two).sin * ((x - y) / two).sin) = x.cos - y.cos
}

// Triple-angle formulas.

/// The sine of three times an angle in terms of sine and cosine.
theorem sin_three(x: Real) {
    (two * x + x).sin = three * x.sin * x.cos.pow(Nat.2) - x.sin.pow(Nat.3)
} by {
    sin_add(two * x, x)
    (two * x + x).sin = (two * x).sin * x.cos + (two * x).cos * x.sin
    sin_two(x)
    (two * x).sin = two * x.sin * x.cos
    cos_two(x)
    (two * x).cos = x.cos.pow(Nat.2) - x.sin.pow(Nat.2)
    (two * x + x).sin = (two * x.sin * x.cos) * x.cos + (x.cos.pow(Nat.2) - x.sin.pow(Nat.2)) * x.sin
    pow_suc(x.cos, Nat.1)
    x.cos.pow(Nat.2) = x.cos * x.cos
    pow_suc(x.sin, Nat.1)
    x.sin.pow(Nat.2) = x.sin * x.sin
    (two * x.sin * x.cos) * x.cos = two * x.sin * x.cos.pow(Nat.2)
    pow_suc(x.sin, Nat.2)
    x.sin.pow(Nat.3) = x.sin * x.sin.pow(Nat.2)
    mul_sub_distrib_left(x.cos.pow(Nat.2), x.sin.pow(Nat.2), x.sin)
    (x.cos.pow(Nat.2) - x.sin.pow(Nat.2)) * x.sin = x.cos.pow(Nat.2) * x.sin - x.sin.pow(Nat.2) * x.sin
    x.sin.pow(Nat.2) * x.sin = x.sin.pow(Nat.3)
    (x.cos.pow(Nat.2) - x.sin.pow(Nat.2)) * x.sin = x.cos.pow(Nat.2) * x.sin - x.sin.pow(Nat.3)
    x.cos.pow(Nat.2) * x.sin = x.sin * x.cos.pow(Nat.2)
    two * x.sin * x.cos.pow(Nat.2) + x.sin * x.cos.pow(Nat.2) = three * x.sin * x.cos.pow(Nat.2)
    two * x.sin * x.cos.pow(Nat.2) + (x.cos.pow(Nat.2) * x.sin - x.sin.pow(Nat.3)) =
        (two * x.sin * x.cos.pow(Nat.2) + x.cos.pow(Nat.2) * x.sin) - x.sin.pow(Nat.3)
    two * x.sin * x.cos.pow(Nat.2) + (x.cos.pow(Nat.2) * x.sin - x.sin.pow(Nat.3)) =
        three * x.sin * x.cos.pow(Nat.2) - x.sin.pow(Nat.3)
    (two * x + x).sin = three * x.sin * x.cos.pow(Nat.2) - x.sin.pow(Nat.3)
}

/// The cosine of three times an angle in terms of sine and cosine.
theorem cos_three(x: Real) {
    (two * x + x).cos = x.cos.pow(Nat.3) - three * x.sin.pow(Nat.2) * x.cos
} by {
    cos_add(two * x, x)
    (two * x + x).cos = (two * x).cos * x.cos - (two * x).sin * x.sin
    cos_two(x)
    (two * x).cos = x.cos.pow(Nat.2) - x.sin.pow(Nat.2)
    sin_two(x)
    (two * x).sin = two * x.sin * x.cos
    (two * x + x).cos = (x.cos.pow(Nat.2) - x.sin.pow(Nat.2)) * x.cos - (two * x.sin * x.cos) * x.sin
    pow_suc(x.cos, Nat.1)
    x.cos.pow(Nat.2) = x.cos * x.cos
    pow_suc(x.sin, Nat.1)
    x.sin.pow(Nat.2) = x.sin * x.sin
    pow_suc(x.cos, Nat.2)
    x.cos.pow(Nat.3) = x.cos * x.cos.pow(Nat.2)
    (x.cos.pow(Nat.2) - x.sin.pow(Nat.2)) * x.cos = x.cos.pow(Nat.3) - x.sin.pow(Nat.2) * x.cos
    (two * x.sin * x.cos) * x.sin = two * x.sin * (x.cos * x.sin)
    two * x.sin * (x.cos * x.sin) = two * x.sin * (x.sin * x.cos)
    two * x.sin * (x.sin * x.cos) = two * (x.sin * x.sin) * x.cos
    two * (x.sin * x.sin) * x.cos = two * x.sin.pow(Nat.2) * x.cos
    (two * x.sin * x.cos) * x.sin = two * x.sin.pow(Nat.2) * x.cos
    one_add_two_mul(x.sin.pow(Nat.2) * x.cos)
    x.sin.pow(Nat.2) * x.cos + two * x.sin.pow(Nat.2) * x.cos = three * x.sin.pow(Nat.2) * x.cos
    x.cos.pow(Nat.3) - x.sin.pow(Nat.2) * x.cos - two * x.sin.pow(Nat.2) * x.cos =
        x.cos.pow(Nat.3) - (x.sin.pow(Nat.2) * x.cos + two * x.sin.pow(Nat.2) * x.cos)
    x.cos.pow(Nat.3) - (x.sin.pow(Nat.2) * x.cos + two * x.sin.pow(Nat.2) * x.cos) =
        x.cos.pow(Nat.3) - three * x.sin.pow(Nat.2) * x.cos
    x.cos.pow(Nat.3) - x.sin.pow(Nat.2) * x.cos - two * x.sin.pow(Nat.2) * x.cos =
        x.cos.pow(Nat.3) - three * x.sin.pow(Nat.2) * x.cos
    (two * x + x).cos = x.cos.pow(Nat.3) - three * x.sin.pow(Nat.2) * x.cos
}

/// The sine of three times an angle in terms of sine alone.
theorem sin_three_sin(x: Real) {
    (two * x + x).sin = three * x.sin - four * x.sin.pow(Nat.3)
} by {
    sin_three(x)
    (two * x + x).sin = three * x.sin * x.cos.pow(Nat.2) - x.sin.pow(Nat.3)
    sin_sq_add_cos_sq(x)
    x.sin.pow(Nat.2) + x.cos.pow(Nat.2) = Real.1
    x.cos.pow(Nat.2) = Real.1 - x.sin.pow(Nat.2)
    pow_suc(x.sin, Nat.2)
    x.sin.pow(Nat.3) = x.sin * x.sin.pow(Nat.2)
    three * x.sin * x.cos.pow(Nat.2) = three * x.sin * (Real.1 - x.sin.pow(Nat.2))
    three * x.sin * (Real.1 - x.sin.pow(Nat.2)) = three * x.sin - three * x.sin * x.sin.pow(Nat.2)
    three * x.sin * x.sin.pow(Nat.2) = three * x.sin.pow(Nat.3)
    three * x.sin * x.cos.pow(Nat.2) = three * x.sin - three * x.sin.pow(Nat.3)
    three * x.sin * x.cos.pow(Nat.2) - x.sin.pow(Nat.3) =
        three * x.sin - three * x.sin.pow(Nat.3) - x.sin.pow(Nat.3)
    three_add_one_mul(x.sin.pow(Nat.3))
    three * x.sin.pow(Nat.3) + x.sin.pow(Nat.3) = four * x.sin.pow(Nat.3)
    three * x.sin - three * x.sin.pow(Nat.3) - x.sin.pow(Nat.3) =
        three * x.sin - (three * x.sin.pow(Nat.3) + x.sin.pow(Nat.3))
    three * x.sin - (three * x.sin.pow(Nat.3) + x.sin.pow(Nat.3)) =
        three * x.sin - four * x.sin.pow(Nat.3)
    three * x.sin - three * x.sin.pow(Nat.3) - x.sin.pow(Nat.3) = three * x.sin - four * x.sin.pow(Nat.3)
    (two * x + x).sin = three * x.sin - four * x.sin.pow(Nat.3)
}

/// The cosine of three times an angle in terms of cosine alone.
theorem cos_three_cos(x: Real) {
    (two * x + x).cos = four * x.cos.pow(Nat.3) - three * x.cos
} by {
    cos_three(x)
    (two * x + x).cos = x.cos.pow(Nat.3) - three * x.sin.pow(Nat.2) * x.cos
    sin_sq_add_cos_sq(x)
    x.sin.pow(Nat.2) + x.cos.pow(Nat.2) = Real.1
    x.sin.pow(Nat.2) = Real.1 - x.cos.pow(Nat.2)
    pow_suc(x.cos, Nat.2)
    x.cos.pow(Nat.3) = x.cos * x.cos.pow(Nat.2)
    three * x.sin.pow(Nat.2) * x.cos = three * (Real.1 - x.cos.pow(Nat.2)) * x.cos
    three * (Real.1 - x.cos.pow(Nat.2)) * x.cos = three * x.cos - three * x.cos.pow(Nat.2) * x.cos
    three * x.cos.pow(Nat.2) * x.cos = three * x.cos.pow(Nat.3)
    three * x.sin.pow(Nat.2) * x.cos = three * x.cos - three * x.cos.pow(Nat.3)
    x.cos.pow(Nat.3) - three * x.sin.pow(Nat.2) * x.cos =
        x.cos.pow(Nat.3) - (three * x.cos - three * x.cos.pow(Nat.3))
    sub_diff_add(x.cos.pow(Nat.3), three * x.cos, three * x.cos.pow(Nat.3))
    x.cos.pow(Nat.3) - (three * x.cos - three * x.cos.pow(Nat.3)) =
        x.cos.pow(Nat.3) - three * x.cos + three * x.cos.pow(Nat.3)
    x.cos.pow(Nat.3) - three * x.cos + three * x.cos.pow(Nat.3) =
        x.cos.pow(Nat.3) + three * x.cos.pow(Nat.3) - three * x.cos
    one_add_three_mul(x.cos.pow(Nat.3))
    x.cos.pow(Nat.3) + three * x.cos.pow(Nat.3) = four * x.cos.pow(Nat.3)
    x.cos.pow(Nat.3) + three * x.cos.pow(Nat.3) - three * x.cos = four * x.cos.pow(Nat.3) - three * x.cos
    x.cos.pow(Nat.3) - three * x.cos + three * x.cos.pow(Nat.3) = four * x.cos.pow(Nat.3) - three * x.cos
    x.cos.pow(Nat.3) - three * x.sin.pow(Nat.2) * x.cos = four * x.cos.pow(Nat.3) - three * x.cos
    (two * x + x).cos = four * x.cos.pow(Nat.3) - three * x.cos
}

// Half-angle applications of the double-angle formulas.

/// Cosine in terms of the square of the cosine of the half-angle.
theorem cos_half_sq(x: Real) {
    x.cos = two * (x / two).cos.pow(Nat.2) - Real.1
} by {
    cos_two_alt1(x / two)
    (two * (x / two)).cos = two * (x / two).cos.pow(Nat.2) - Real.1
    mul_div_cancel(x, two)
    two * (x / two) = x
    (two * (x / two)).cos = x.cos
    x.cos = two * (x / two).cos.pow(Nat.2) - Real.1
}

/// Sine in terms of the product of sine and cosine of the half-angle.
theorem sin_half_sq(x: Real) {
    x.sin = two * (x / two).sin * (x / two).cos
} by {
    sin_two(x / two)
    (two * (x / two)).sin = two * (x / two).sin * (x / two).cos
    mul_div_cancel(x, two)
    two * (x / two) = x
    (two * (x / two)).sin = x.sin
    x.sin = two * (x / two).sin * (x / two).cos
}

/// Cosine in terms of the square of the sine of the half-angle.
theorem cos_half_sin(x: Real) {
    x.cos = Real.1 - two * (x / two).sin.pow(Nat.2)
} by {
    cos_two_alt2(x / two)
    (two * (x / two)).cos = Real.1 - two * (x / two).sin.pow(Nat.2)
    mul_div_cancel(x, two)
    two * (x / two) = x
    (two * (x / two)).cos = x.cos
    x.cos = Real.1 - two * (x / two).sin.pow(Nat.2)
}

// Parity combinations.

/// The square of sine at -x is the square at x.
theorem sin_sq_neg(x: Real) {
    (-x).sin.pow(Nat.2) = x.sin.pow(Nat.2)
} by {
    sin_neg(x)
    (-x).sin = -x.sin
    (-x).sin.pow(Nat.2) = (-x.sin).pow(Nat.2)
    pow_suc(-x.sin, Nat.1)
    (-x.sin).pow(Nat.2) = (-x.sin) * (-x.sin)
    neg_mul_neg(x.sin, x.sin)
    (-x.sin) * (-x.sin) = x.sin * x.sin
    pow_suc(x.sin, Nat.1)
    x.sin.pow(Nat.2) = x.sin * x.sin
    (-x.sin).pow(Nat.2) = x.sin.pow(Nat.2)
    (-x).sin.pow(Nat.2) = x.sin.pow(Nat.2)
}

/// The square of cosine at -x is the square at x.
theorem cos_sq_neg(x: Real) {
    (-x).cos.pow(Nat.2) = x.cos.pow(Nat.2)
} by {
    cos_neg(x)
    (-x).cos = x.cos
    (-x).cos.pow(Nat.2) = x.cos.pow(Nat.2)
}

/// The product of sine and cosine at -x is the negative of the product at x.
theorem sin_neg_mul_cos_neg(x: Real) {
    (-x).sin * (-x).cos = -x.sin * x.cos
} by {
    sin_neg(x)
    (-x).sin = -x.sin
    cos_neg(x)
    (-x).cos = x.cos
    (-x).sin * (-x).cos = (-x.sin) * x.cos
    (-x.sin) * x.cos = -x.sin * x.cos
    (-x).sin * (-x).cos = -x.sin * x.cos
}

/// The product of sine and cosine of -x is the product at x.
theorem sin_mul_cos_neg(x: Real) {
    x.sin * (-x).cos = x.sin * x.cos
} by {
    cos_neg(x)
    (-x).cos = x.cos
    x.sin * (-x).cos = x.sin * x.cos
}

// Tangent and the reciprocal trigonometric functions.
//
// The tangent is defined as sine over cosine; secant, cosecant and cotangent
// are the usual reciprocals.  The identities below are mostly conditional on
// the relevant denominators being nonzero.

/// The tangent function: sine over cosine.
define tan(x: Real) -> Real {
    x.sin / x.cos
}

/// The secant function: the reciprocal of cosine.
define sec(x: Real) -> Real {
    Real.1 / x.cos
}

/// The cosecant function: the reciprocal of sine.
define csc(x: Real) -> Real {
    Real.1 / x.sin
}

/// The cotangent function: cosine over sine.
define cot(x: Real) -> Real {
    x.cos / x.sin
}

/// Negative one is not zero.
theorem neg_one_ne_zero {
    -Real.1 != Real.0
} by {
    if -Real.1 = Real.0 {
        --Real.1 = -Real.0
        neg_neg(Real.1)
        --Real.1 = Real.1
        neg_zero
        -Real.0 = Real.0
        Real.1 = Real.0
        zero_is_different_than_one
        Real.0 != Real.1
        false
    }
}

/// The tangent of zero is zero.
theorem tan_zero {
    tan(Real.0) = Real.0
} by {
    tan(Real.0) = (Real.0).sin / (Real.0).cos
    sin_zero
    (Real.0).sin = Real.0
    cos_zero
    (Real.0).cos = Real.1
    (Real.0).sin / (Real.0).cos = Real.0 / Real.1
    Real.0 / Real.1 = Real.0
    tan(Real.0) = Real.0
}

/// The tangent of pi is zero.
theorem tan_pi {
    tan(pi) = Real.0
} by {
    tan(pi) = pi.sin / pi.cos
    sin_pi_zero
    pi.sin = Real.0
    cos_pi_neg_one
    pi.cos = -Real.1
    pi.sin / pi.cos = Real.0 / -Real.1
    Real.0 / -Real.1 = Real.0
    tan(pi) = Real.0
}

/// Tangent is odd: tan(-x) = -tan(x).
theorem tan_neg(x: Real) {
    tan(-x) = -tan(x)
} by {
    tan(-x) = (-x).sin / (-x).cos
    sin_neg(x)
    (-x).sin = -x.sin
    cos_neg(x)
    (-x).cos = x.cos
    (-x).sin / (-x).cos = (-x.sin) / x.cos
    (-x.sin) / x.cos = -(x.sin / x.cos)
    tan(x) = x.sin / x.cos
    -(x.sin / x.cos) = -tan(x)
    tan(-x) = -tan(x)
}

/// The tangent of a sum as a single fraction.
theorem tan_add_basic(x: Real, y: Real) {
    tan(x + y) = (x.sin * y.cos + x.cos * y.sin) / (x.cos * y.cos - x.sin * y.sin)
} by {
    tan(x + y) = (x + y).sin / (x + y).cos
    sin_add(x, y)
    (x + y).sin = x.sin * y.cos + x.cos * y.sin
    cos_add(x, y)
    (x + y).cos = x.cos * y.cos - x.sin * y.sin
    (x + y).sin / (x + y).cos = (x.sin * y.cos + x.cos * y.sin) / (x.cos * y.cos - x.sin * y.sin)
    tan(x + y) = (x.sin * y.cos + x.cos * y.sin) / (x.cos * y.cos - x.sin * y.sin)
}

/// Tangent times cosine recovers sine.
theorem tan_mul_cos(x: Real) {
    x.cos != Real.0 implies tan(x) * x.cos = x.sin
} by {
    if x.cos != Real.0 {
        tan(x) = x.sin / x.cos
        tan(x) * x.cos = (x.sin / x.cos) * x.cos
        (x.sin / x.cos) * x.cos = x.cos * (x.sin / x.cos)
        mul_div_cancel(x.sin, x.cos)
        x.cos * (x.sin / x.cos) = x.sin
        tan(x) * x.cos = x.sin
    }
}

/// The sine of a sum in terms of tangents.
theorem sin_add_tan(x: Real, y: Real) {
    x.cos != Real.0 and y.cos != Real.0
    implies
    (x + y).sin = x.cos * y.cos * (tan(x) + tan(y))
} by {
    if x.cos != Real.0 and y.cos != Real.0 {
        sin_add(x, y)
        (x + y).sin = x.sin * y.cos + x.cos * y.sin
        tan_mul_cos(x)
        tan(x) * x.cos = x.sin
        tan_mul_cos(y)
        tan(y) * y.cos = y.sin
        x.sin * y.cos + x.cos * y.sin = (tan(x) * x.cos) * y.cos + x.cos * y.sin
        x.sin * y.cos + x.cos * y.sin = (tan(x) * x.cos) * y.cos + x.cos * (tan(y) * y.cos)
        (tan(x) * x.cos) * y.cos = tan(x) * (x.cos * y.cos)
        x.cos * (tan(y) * y.cos) = tan(y) * (x.cos * y.cos)
        tan(x) * (x.cos * y.cos) + tan(y) * (x.cos * y.cos) = (tan(x) + tan(y)) * (x.cos * y.cos)
        (tan(x) + tan(y)) * (x.cos * y.cos) = x.cos * y.cos * (tan(x) + tan(y))
        x.sin * y.cos + x.cos * y.sin = x.cos * y.cos * (tan(x) + tan(y))
        (x + y).sin = x.cos * y.cos * (tan(x) + tan(y))
    }
}

/// The cosine of a sum in terms of tangents.
theorem cos_add_tan(x: Real, y: Real) {
    x.cos != Real.0 and y.cos != Real.0
    implies
    (x + y).cos = x.cos * y.cos * (Real.1 - tan(x) * tan(y))
} by {
    if x.cos != Real.0 and y.cos != Real.0 {
        cos_add(x, y)
        (x + y).cos = x.cos * y.cos - x.sin * y.sin
        tan_mul_cos(x)
        tan(x) * x.cos = x.sin
        tan_mul_cos(y)
        tan(y) * y.cos = y.sin
        x.cos * y.cos - x.sin * y.sin = x.cos * y.cos - (tan(x) * x.cos) * y.sin
        x.cos * y.cos - x.sin * y.sin = x.cos * y.cos - (tan(x) * x.cos) * (tan(y) * y.cos)
        signed_mul_combine(tan(x), tan(y), x.cos, y.cos)
        (tan(x) * x.cos) * (tan(y) * y.cos) = (tan(x) * tan(y)) * (x.cos * y.cos)
        x.cos * y.cos - (tan(x) * x.cos) * (tan(y) * y.cos) =
            x.cos * y.cos - (tan(x) * tan(y)) * (x.cos * y.cos)
        x.cos * y.cos - (tan(x) * tan(y)) * (x.cos * y.cos) = x.cos * y.cos * (Real.1 - tan(x) * tan(y))
        (x + y).cos = x.cos * y.cos * (Real.1 - tan(x) * tan(y))
    }
}

/// The tangent addition formula.
theorem tan_add(x: Real, y: Real) {
    x.cos != Real.0 and y.cos != Real.0 and Real.1 - tan(x) * tan(y) != Real.0
    implies
    tan(x + y) = (tan(x) + tan(y)) / (Real.1 - tan(x) * tan(y))
} by {
    if x.cos != Real.0 and y.cos != Real.0 and Real.1 - tan(x) * tan(y) != Real.0 {
        tan_add_basic(x, y)
        tan(x + y) = (x.sin * y.cos + x.cos * y.sin) / (x.cos * y.cos - x.sin * y.sin)
        sin_add_tan(x, y)
        (x + y).sin = x.cos * y.cos * (tan(x) + tan(y))
        sin_add(x, y)
        (x + y).sin = x.sin * y.cos + x.cos * y.sin
        x.sin * y.cos + x.cos * y.sin = x.cos * y.cos * (tan(x) + tan(y))
        cos_add_tan(x, y)
        (x + y).cos = x.cos * y.cos * (Real.1 - tan(x) * tan(y))
        cos_add(x, y)
        (x + y).cos = x.cos * y.cos - x.sin * y.sin
        x.cos * y.cos - x.sin * y.sin = x.cos * y.cos * (Real.1 - tan(x) * tan(y))
        tan(x + y) = (x.cos * y.cos * (tan(x) + tan(y))) / (x.cos * y.cos * (Real.1 - tan(x) * tan(y)))
        mul_not_zero[Real](x.cos, y.cos)
        x.cos != Real.0 and y.cos != Real.0 implies x.cos * y.cos != Real.0
        x.cos * y.cos != Real.0
        div_cancel_common(tan(x) + tan(y), x.cos * y.cos, Real.1 - tan(x) * tan(y))
        ((tan(x) + tan(y)) * (x.cos * y.cos)) / ((Real.1 - tan(x) * tan(y)) * (x.cos * y.cos)) =
            (tan(x) + tan(y)) / (Real.1 - tan(x) * tan(y))
        (x.cos * y.cos * (tan(x) + tan(y))) / (x.cos * y.cos * (Real.1 - tan(x) * tan(y))) =
            ((tan(x) + tan(y)) * (x.cos * y.cos)) / ((Real.1 - tan(x) * tan(y)) * (x.cos * y.cos))
        tan(x + y) = (tan(x) + tan(y)) / (Real.1 - tan(x) * tan(y))
    }
}

/// Pi is a period of the tangent.
theorem tan_add_pi(x: Real) {
    x.cos != Real.0 implies tan(x + pi) = tan(x)
} by {
    if x.cos != Real.0 {
        cos_pi_neg_one
        pi.cos = -Real.1
        neg_one_ne_zero
        -Real.1 != Real.0
        pi.cos != Real.0
        tan_pi
        tan(pi) = Real.0
        Real.1 - tan(x) * tan(pi) = Real.1 - tan(x) * Real.0
        tan(x) * Real.0 = Real.0
        Real.1 - tan(x) * Real.0 = Real.1 - Real.0
        Real.1 - Real.0 = Real.1
        Real.1 - tan(x) * tan(pi) = Real.1
        zero_is_different_than_one
        Real.0 != Real.1
        Real.1 - tan(x) * tan(pi) != Real.0
        tan_add(x, pi)
        tan(x + pi) = (tan(x) + tan(pi)) / (Real.1 - tan(x) * tan(pi))
        (tan(x) + tan(pi)) / (Real.1 - tan(x) * tan(pi)) = (tan(x) + Real.0) / (Real.1 - tan(x) * Real.0)
        tan(x) + Real.0 = tan(x)
        Real.1 - tan(x) * Real.0 = Real.1 - Real.0
        Real.1 - Real.0 = Real.1
        (tan(x) + Real.0) / (Real.1 - tan(x) * Real.0) = tan(x) / Real.1
        tan(x) / Real.1 = tan(x)
        tan(x + pi) = tan(x)
    }
}

/// The tangent of twice an angle.
theorem tan_two(x: Real) {
    x.cos != Real.0 and Real.1 - tan(x).pow(Nat.2) != Real.0
    implies
    tan(two * x) = two * tan(x) / (Real.1 - tan(x).pow(Nat.2))
} by {
    if x.cos != Real.0 and Real.1 - tan(x).pow(Nat.2) != Real.0 {
        pow_suc(tan(x), Nat.1)
        tan(x).pow(Nat.2) = tan(x) * tan(x)
        Real.1 - tan(x) * tan(x) != Real.0
        tan_add(x, x)
        tan(x + x) = (tan(x) + tan(x)) / (Real.1 - tan(x) * tan(x))
        two = Real.1 + Real.1
        two * x = x + x
        tan(two * x) = tan(x + x)
        tan(x) + tan(x) = two * tan(x)
        (tan(x) + tan(x)) / (Real.1 - tan(x) * tan(x)) = two * tan(x) / (Real.1 - tan(x) * tan(x))
        two * tan(x) / (Real.1 - tan(x) * tan(x)) = two * tan(x) / (Real.1 - tan(x).pow(Nat.2))
        tan(two * x) = two * tan(x) / (Real.1 - tan(x).pow(Nat.2))
    }
}

/// Secant times cosine is one.
theorem sec_mul_cos(x: Real) {
    x.cos != Real.0 implies sec(x) * x.cos = Real.1
} by {
    if x.cos != Real.0 {
        sec(x) = Real.1 / x.cos
        sec(x) * x.cos = (Real.1 / x.cos) * x.cos
        (Real.1 / x.cos) * x.cos = x.cos * (Real.1 / x.cos)
        mul_div_cancel(Real.1, x.cos)
        x.cos * (Real.1 / x.cos) = Real.1
        sec(x) * x.cos = Real.1
    }
}

/// Cosecant times sine is one.
theorem csc_mul_sin(x: Real) {
    x.sin != Real.0 implies csc(x) * x.sin = Real.1
} by {
    if x.sin != Real.0 {
        csc(x) = Real.1 / x.sin
        csc(x) * x.sin = (Real.1 / x.sin) * x.sin
        (Real.1 / x.sin) * x.sin = x.sin * (Real.1 / x.sin)
        mul_div_cancel(Real.1, x.sin)
        x.sin * (Real.1 / x.sin) = Real.1
        csc(x) * x.sin = Real.1
    }
}

/// Tangent times cotangent is one.
theorem tan_mul_cot(x: Real) {
    x.sin != Real.0 and x.cos != Real.0 implies tan(x) * cot(x) = Real.1
} by {
    if x.sin != Real.0 and x.cos != Real.0 {
        tan(x) = x.sin / x.cos
        cot(x) = x.cos / x.sin
        tan(x) * cot(x) = (x.sin / x.cos) * (x.cos / x.sin)
        mul_div(x.sin, x.cos, x.cos, x.sin)
        (x.sin / x.cos) * (x.cos / x.sin) = (x.sin * x.cos) / (x.cos * x.sin)
        x.sin * x.cos = x.cos * x.sin
        (x.sin * x.cos) / (x.cos * x.sin) = (x.cos * x.sin) / (x.cos * x.sin)
        mul_not_zero[Real](x.cos, x.sin)
        x.cos != Real.0 and x.sin != Real.0 implies x.cos * x.sin != Real.0
        x.cos * x.sin != Real.0
        div_mul_cancel_left(x.cos * x.sin, Real.1)
        ((x.cos * x.sin) * Real.1) / (x.cos * x.sin) = Real.1
        (x.cos * x.sin) / (x.cos * x.sin) = Real.1
        tan(x) * cot(x) = Real.1
    }
}

/// The square of secant minus the square of tangent is one.
theorem sec_sq_sub_tan_sq(x: Real) {
    x.cos != Real.0 implies sec(x).pow(Nat.2) - tan(x).pow(Nat.2) = Real.1
} by {
    if x.cos != Real.0 {
        sin_sq_add_cos_sq(x)
        x.sin.pow(Nat.2) + x.cos.pow(Nat.2) = Real.1
        sec(x) = Real.1 / x.cos
        tan(x) = x.sin / x.cos
        pow_suc(sec(x), Nat.1)
        sec(x).pow(Nat.2) = sec(x) * sec(x)
        pow_suc(tan(x), Nat.1)
        tan(x).pow(Nat.2) = tan(x) * tan(x)
        pow_suc(x.sin, Nat.1)
        x.sin.pow(Nat.2) = x.sin * x.sin
        pow_suc(x.cos, Nat.1)
        x.cos.pow(Nat.2) = x.cos * x.cos
        sec(x).pow(Nat.2) - tan(x).pow(Nat.2) = (Real.1 / x.cos) * (Real.1 / x.cos) - (x.sin / x.cos) * (x.sin / x.cos)
        mul_div(Real.1, x.cos, Real.1, x.cos)
        (Real.1 / x.cos) * (Real.1 / x.cos) = (Real.1 * Real.1) / (x.cos * x.cos)
        mul_div(x.sin, x.cos, x.sin, x.cos)
        (x.sin / x.cos) * (x.sin / x.cos) = (x.sin * x.sin) / (x.cos * x.cos)
        (Real.1 / x.cos) * (Real.1 / x.cos) - (x.sin / x.cos) * (x.sin / x.cos) =
            (Real.1 * Real.1) / (x.cos * x.cos) - (x.sin * x.sin) / (x.cos * x.cos)
        mul_not_zero[Real](x.cos, x.cos)
        x.cos != Real.0 and x.cos != Real.0 implies x.cos * x.cos != Real.0
        x.cos * x.cos != Real.0
        div_sub_same_denom(Real.1, Real.1, Real.1, x.sin * x.sin, x.cos * x.cos)
        Real.1 * Real.1 / (x.cos * x.cos) - Real.1 * (x.sin * x.sin) / (x.cos * x.cos) =
            (Real.1 * Real.1 - Real.1 * (x.sin * x.sin)) / (x.cos * x.cos)
        Real.1 * Real.1 / (x.cos * x.cos) - Real.1 * (x.sin * x.sin) / (x.cos * x.cos) =
            (Real.1 * Real.1) / (x.cos * x.cos) - (x.sin * x.sin) / (x.cos * x.cos)
        Real.1 * Real.1 - Real.1 * (x.sin * x.sin) = Real.1 - x.sin * x.sin
        (Real.1 * Real.1 - Real.1 * (x.sin * x.sin)) / (x.cos * x.cos) =
            (Real.1 - x.sin * x.sin) / (x.cos * x.cos)
        x.sin * x.sin = x.sin.pow(Nat.2)
        (Real.1 - x.sin * x.sin) / (x.cos * x.cos) = (Real.1 - x.sin.pow(Nat.2)) / (x.cos * x.cos)
        (Real.1 - x.sin.pow(Nat.2)) / (x.cos * x.cos) = (Real.1 - x.sin.pow(Nat.2)) / x.cos.pow(Nat.2)
        x.cos.pow(Nat.2) = Real.1 - x.sin.pow(Nat.2)
        (Real.1 - x.sin.pow(Nat.2)) / x.cos.pow(Nat.2) = x.cos.pow(Nat.2) / x.cos.pow(Nat.2)
        x.cos.pow(Nat.2) != Real.0
        div_mul_cancel_left(x.cos.pow(Nat.2), Real.1)
        ((x.cos.pow(Nat.2) * Real.1) / x.cos.pow(Nat.2)) = Real.1
        x.cos.pow(Nat.2) / x.cos.pow(Nat.2) = Real.1
        sec(x).pow(Nat.2) - tan(x).pow(Nat.2) = Real.1
    }
}

/// The square of cosecant minus the square of cotangent is one.
theorem csc_sq_sub_cot_sq(x: Real) {
    x.sin != Real.0 implies csc(x).pow(Nat.2) - cot(x).pow(Nat.2) = Real.1
} by {
    if x.sin != Real.0 {
        sin_sq_add_cos_sq(x)
        x.sin.pow(Nat.2) + x.cos.pow(Nat.2) = Real.1
        csc(x) = Real.1 / x.sin
        cot(x) = x.cos / x.sin
        pow_suc(csc(x), Nat.1)
        csc(x).pow(Nat.2) = csc(x) * csc(x)
        pow_suc(cot(x), Nat.1)
        cot(x).pow(Nat.2) = cot(x) * cot(x)
        pow_suc(x.cos, Nat.1)
        x.cos.pow(Nat.2) = x.cos * x.cos
        pow_suc(x.sin, Nat.1)
        x.sin.pow(Nat.2) = x.sin * x.sin
        csc(x).pow(Nat.2) - cot(x).pow(Nat.2) = (Real.1 / x.sin) * (Real.1 / x.sin) - (x.cos / x.sin) * (x.cos / x.sin)
        mul_div(Real.1, x.sin, Real.1, x.sin)
        (Real.1 / x.sin) * (Real.1 / x.sin) = (Real.1 * Real.1) / (x.sin * x.sin)
        mul_div(x.cos, x.sin, x.cos, x.sin)
        (x.cos / x.sin) * (x.cos / x.sin) = (x.cos * x.cos) / (x.sin * x.sin)
        (Real.1 / x.sin) * (Real.1 / x.sin) - (x.cos / x.sin) * (x.cos / x.sin) =
            (Real.1 * Real.1) / (x.sin * x.sin) - (x.cos * x.cos) / (x.sin * x.sin)
        mul_not_zero[Real](x.sin, x.sin)
        x.sin != Real.0 and x.sin != Real.0 implies x.sin * x.sin != Real.0
        x.sin * x.sin != Real.0
        div_sub_same_denom(Real.1, Real.1, Real.1, x.cos * x.cos, x.sin * x.sin)
        Real.1 * Real.1 / (x.sin * x.sin) - Real.1 * (x.cos * x.cos) / (x.sin * x.sin) =
            (Real.1 * Real.1 - Real.1 * (x.cos * x.cos)) / (x.sin * x.sin)
        Real.1 * Real.1 / (x.sin * x.sin) - Real.1 * (x.cos * x.cos) / (x.sin * x.sin) =
            (Real.1 * Real.1) / (x.sin * x.sin) - (x.cos * x.cos) / (x.sin * x.sin)
        Real.1 * Real.1 - Real.1 * (x.cos * x.cos) = Real.1 - x.cos * x.cos
        (Real.1 * Real.1 - Real.1 * (x.cos * x.cos)) / (x.sin * x.sin) =
            (Real.1 - x.cos * x.cos) / (x.sin * x.sin)
        x.cos * x.cos = x.cos.pow(Nat.2)
        (Real.1 - x.cos * x.cos) / (x.sin * x.sin) = (Real.1 - x.cos.pow(Nat.2)) / (x.sin * x.sin)
        (Real.1 - x.cos.pow(Nat.2)) / (x.sin * x.sin) = (Real.1 - x.cos.pow(Nat.2)) / x.sin.pow(Nat.2)
        x.sin.pow(Nat.2) = Real.1 - x.cos.pow(Nat.2)
        (Real.1 - x.cos.pow(Nat.2)) / x.sin.pow(Nat.2) = x.sin.pow(Nat.2) / x.sin.pow(Nat.2)
        x.sin.pow(Nat.2) != Real.0
        div_mul_cancel_left(x.sin.pow(Nat.2), Real.1)
        ((x.sin.pow(Nat.2) * Real.1) / x.sin.pow(Nat.2)) = Real.1
        x.sin.pow(Nat.2) / x.sin.pow(Nat.2) = Real.1
        csc(x).pow(Nat.2) - cot(x).pow(Nat.2) = Real.1
    }
}

// Special value at pi over four.
//
// Pi over four is not predefined, so we define it as half of pi over two and
// derive the squared sine and cosine values, their equality, and the tangent.
// The exact value (2).sqrt/2 is out of reach because sqrt is Option-valued.

/// Pi over four as a real number.
let pi_over_four = pi_over_two / two

/// The reciprocal of two is not zero.
theorem real_one_over_two_ne_zero {
    Real.1 / two != Real.0
} by {
    two_nonzero
    two != Real.0
    mul_div_cancel(Real.1, two)
    two * (Real.1 / two) = Real.1
    zero_is_different_than_one
    Real.0 != Real.1
    two * (Real.1 / two) != Real.0
    if Real.1 / two = Real.0 {
        two * (Real.1 / two) = two * Real.0
        two * Real.0 = Real.0
        two * (Real.1 / two) = Real.0
        two * (Real.1 / two) != Real.0
        false
    }
}

/// The square of the cosine of pi over four is one half.
theorem cos_sq_pi_over_four {
    pi_over_four.cos.pow(Nat.2) = Real.1 / two
} by {
    cos_two_alt1(pi_over_four)
    (two * pi_over_four).cos = two * pi_over_four.cos.pow(Nat.2) - Real.1
    mul_div_cancel(pi_over_two, two)
    two * (pi_over_two / two) = pi_over_two
    pi_over_four = pi_over_two / two
    two * pi_over_four = two * (pi_over_two / two)
    two * pi_over_four = pi_over_two
    (two * pi_over_four).cos = pi_over_two.cos
    cos_pi_over_two_zero
    pi_over_two.cos = Real.0
    two * pi_over_four.cos.pow(Nat.2) - Real.1 = Real.0
    two * pi_over_four.cos.pow(Nat.2) = Real.1
    (two * pi_over_four.cos.pow(Nat.2)) / two = Real.1 / two
    div_mul_cancel_left(two, pi_over_four.cos.pow(Nat.2))
    (two * pi_over_four.cos.pow(Nat.2)) / two = pi_over_four.cos.pow(Nat.2)
    pi_over_four.cos.pow(Nat.2) = Real.1 / two
}

/// The square of the sine of pi over four is one half.
theorem sin_sq_pi_over_four {
    pi_over_four.sin.pow(Nat.2) = Real.1 / two
} by {
    cos_two_alt2(pi_over_four)
    (two * pi_over_four).cos = Real.1 - two * pi_over_four.sin.pow(Nat.2)
    mul_div_cancel(pi_over_two, two)
    two * (pi_over_two / two) = pi_over_two
    pi_over_four = pi_over_two / two
    two * pi_over_four = two * (pi_over_two / two)
    two * pi_over_four = pi_over_two
    (two * pi_over_four).cos = pi_over_two.cos
    cos_pi_over_two_zero
    pi_over_two.cos = Real.0
    Real.1 - two * pi_over_four.sin.pow(Nat.2) = Real.0
    two * pi_over_four.sin.pow(Nat.2) = Real.1
    (two * pi_over_four.sin.pow(Nat.2)) / two = Real.1 / two
    div_mul_cancel_left(two, pi_over_four.sin.pow(Nat.2))
    (two * pi_over_four.sin.pow(Nat.2)) / two = pi_over_four.sin.pow(Nat.2)
    pi_over_four.sin.pow(Nat.2) = Real.1 / two
}

/// The product of sine and cosine of pi over four is one half.
theorem sin_cos_pi_over_four {
    pi_over_four.sin * pi_over_four.cos = Real.1 / two
} by {
    sin_two(pi_over_four)
    (two * pi_over_four).sin = two * pi_over_four.sin * pi_over_four.cos
    mul_div_cancel(pi_over_two, two)
    two * (pi_over_two / two) = pi_over_two
    pi_over_four = pi_over_two / two
    two * pi_over_four = two * (pi_over_two / two)
    two * pi_over_four = pi_over_two
    (two * pi_over_four).sin = pi_over_two.sin
    sin_pi_over_two_one
    pi_over_two.sin = Real.1
    two * pi_over_four.sin * pi_over_four.cos = Real.1
    (two * pi_over_four.sin * pi_over_four.cos) / two = Real.1 / two
    div_mul_cancel_left(two, pi_over_four.sin * pi_over_four.cos)
    (two * (pi_over_four.sin * pi_over_four.cos)) / two = pi_over_four.sin * pi_over_four.cos
    two * pi_over_four.sin * pi_over_four.cos = two * (pi_over_four.sin * pi_over_four.cos)
    pi_over_four.sin * pi_over_four.cos = Real.1 / two
}

/// Sine and cosine of pi over four are equal.
theorem sin_eq_cos_pi_over_four {
    pi_over_four.sin = pi_over_four.cos
} by {
    sin_pi_over_two_sub(pi_over_four)
    (pi_over_two - pi_over_four).sin = pi_over_four.cos
    pi_over_four = pi_over_two / two
    mul_div_cancel(pi_over_two, two)
    two * (pi_over_two / two) = pi_over_two
    two * pi_over_four = pi_over_two
    pi_over_two - pi_over_four = two * pi_over_four - pi_over_four
    pi_over_four + pi_over_four = two * pi_over_four
    two * pi_over_four - pi_over_four = pi_over_four + pi_over_four - pi_over_four
    sub_cancels(pi_over_four, pi_over_four)
    pi_over_four + pi_over_four - pi_over_four = pi_over_four
    two * pi_over_four - pi_over_four = pi_over_four
    pi_over_two - pi_over_four = pi_over_four
    (pi_over_two - pi_over_four).sin = pi_over_four.sin
    pi_over_four.sin = pi_over_four.cos
}

/// The cosine of pi over four is not zero.
theorem cos_pi_over_four_ne_zero {
    pi_over_four.cos != Real.0
} by {
    if pi_over_four.cos = Real.0 {
        pow_suc(pi_over_four.cos, Nat.1)
        pi_over_four.cos.pow(Nat.2) = pi_over_four.cos * pi_over_four.cos
        pi_over_four.cos * pi_over_four.cos = Real.0
        pi_over_four.cos.pow(Nat.2) = Real.0
        cos_sq_pi_over_four
        pi_over_four.cos.pow(Nat.2) = Real.1 / two
        real_one_over_two_ne_zero
        Real.1 / two != Real.0
        Real.0 != Real.1 / two
        false
    }
}

/// The tangent of pi over four is one.
theorem tan_pi_over_four {
    tan(pi_over_four) = Real.1
} by {
    tan(pi_over_four) = pi_over_four.sin / pi_over_four.cos
    sin_eq_cos_pi_over_four
    pi_over_four.sin = pi_over_four.cos
    pi_over_four.sin / pi_over_four.cos = pi_over_four.cos / pi_over_four.cos
    cos_pi_over_four_ne_zero
    pi_over_four.cos != Real.0
    div_mul_cancel_left(pi_over_four.cos, Real.1)
    ((pi_over_four.cos * Real.1) / pi_over_four.cos) = Real.1
    pi_over_four.cos / pi_over_four.cos = Real.1
    tan(pi_over_four) = Real.1
}

// Periodicity in terms of two pi times a natural number.

/// Two pi is a period of cosine: (x + 2pi).cos = x.cos.
theorem cos_add_two_pi_two(x: Real) {
    (x + two * pi).cos = x.cos
} by {
    cos_add_two_pi(x)
    (x + (pi + pi)).cos = x.cos
    two = Real.1 + Real.1
    two * pi = pi + pi
    (x + two * pi).cos = x.cos
}

/// The sine of x plus 2n pi is the sine of x, for every natural n.
theorem sin_add_two_pi_n(x: Real, n: Nat) {
    (x + two * (from_nat[Real](n) * pi)).sin = x.sin
} by {
    define p(k: Nat) -> Bool {
        (x + two * (from_nat[Real](k) * pi)).sin = x.sin
    }
    from_nat_zero[Real]
    from_nat[Real](Nat.0) = Real.0
    from_nat[Real](Nat.0) * pi = Real.0
    two * (from_nat[Real](Nat.0) * pi) = Real.0
    x + two * (from_nat[Real](Nat.0) * pi) = x + Real.0
    x + Real.0 = x
    (x + two * (from_nat[Real](Nat.0) * pi)).sin = x.sin
    p(Nat.0)
    forall(k: Nat) {
        if p(k) {
            from_nat_add[Real](k, Nat.1)
            from_nat[Real](k + Nat.1) = from_nat[Real](k) + from_nat[Real](Nat.1)
            k + Nat.1 = k.suc
            from_nat[Real](k.suc) = from_nat[Real](k) + from_nat[Real](Nat.1)
            from_nat_one[Real]
            from_nat[Real](Nat.1) = Real.1
            from_nat[Real](k.suc) = from_nat[Real](k) + Real.1
            from_nat[Real](k.suc) * pi = (from_nat[Real](k) + Real.1) * pi
            (from_nat[Real](k) + Real.1) * pi = from_nat[Real](k) * pi + pi
            from_nat[Real](k.suc) * pi = from_nat[Real](k) * pi + pi
            two * (from_nat[Real](k.suc) * pi) = two * (from_nat[Real](k) * pi + pi)
            two * (from_nat[Real](k) * pi + pi) = two * (from_nat[Real](k) * pi) + two * pi
            two * (from_nat[Real](k.suc) * pi) = two * (from_nat[Real](k) * pi) + two * pi
            x + two * (from_nat[Real](k.suc) * pi) = x + (two * (from_nat[Real](k) * pi) + two * pi)
            x + (two * (from_nat[Real](k) * pi) + two * pi) = (x + two * (from_nat[Real](k) * pi)) + two * pi
            (x + two * (from_nat[Real](k.suc) * pi)).sin = ((x + two * (from_nat[Real](k) * pi)) + two * pi).sin
            sin_add_two_pi_two(x + two * (from_nat[Real](k) * pi))
            ((x + two * (from_nat[Real](k) * pi)) + two * pi).sin = (x + two * (from_nat[Real](k) * pi)).sin
            p(k)
            (x + two * (from_nat[Real](k) * pi)).sin = x.sin
            (x + two * (from_nat[Real](k.suc) * pi)).sin = x.sin
            p(k.suc)
        }
    }
    p(Nat.0) and forall(k: Nat) { p(k) implies p(k.suc) }
    Nat.induction(p)
    p(n)
}

/// The cosine of x plus 2n pi is the cosine of x, for every natural n.
theorem cos_add_two_pi_n(x: Real, n: Nat) {
    (x + two * (from_nat[Real](n) * pi)).cos = x.cos
} by {
    define p(k: Nat) -> Bool {
        (x + two * (from_nat[Real](k) * pi)).cos = x.cos
    }
    from_nat_zero[Real]
    from_nat[Real](Nat.0) = Real.0
    from_nat[Real](Nat.0) * pi = Real.0
    two * (from_nat[Real](Nat.0) * pi) = Real.0
    x + two * (from_nat[Real](Nat.0) * pi) = x + Real.0
    x + Real.0 = x
    (x + two * (from_nat[Real](Nat.0) * pi)).cos = x.cos
    p(Nat.0)
    forall(k: Nat) {
        if p(k) {
            from_nat_add[Real](k, Nat.1)
            from_nat[Real](k + Nat.1) = from_nat[Real](k) + from_nat[Real](Nat.1)
            k + Nat.1 = k.suc
            from_nat[Real](k.suc) = from_nat[Real](k) + from_nat[Real](Nat.1)
            from_nat_one[Real]
            from_nat[Real](Nat.1) = Real.1
            from_nat[Real](k.suc) = from_nat[Real](k) + Real.1
            from_nat[Real](k.suc) * pi = (from_nat[Real](k) + Real.1) * pi
            (from_nat[Real](k) + Real.1) * pi = from_nat[Real](k) * pi + pi
            from_nat[Real](k.suc) * pi = from_nat[Real](k) * pi + pi
            two * (from_nat[Real](k.suc) * pi) = two * (from_nat[Real](k) * pi + pi)
            two * (from_nat[Real](k) * pi + pi) = two * (from_nat[Real](k) * pi) + two * pi
            two * (from_nat[Real](k.suc) * pi) = two * (from_nat[Real](k) * pi) + two * pi
            x + two * (from_nat[Real](k.suc) * pi) = x + (two * (from_nat[Real](k) * pi) + two * pi)
            x + (two * (from_nat[Real](k) * pi) + two * pi) = (x + two * (from_nat[Real](k) * pi)) + two * pi
            (x + two * (from_nat[Real](k.suc) * pi)).cos = ((x + two * (from_nat[Real](k) * pi)) + two * pi).cos
            cos_add_two_pi_two(x + two * (from_nat[Real](k) * pi))
            ((x + two * (from_nat[Real](k) * pi)) + two * pi).cos = (x + two * (from_nat[Real](k) * pi)).cos
            p(k)
            (x + two * (from_nat[Real](k) * pi)).cos = x.cos
            (x + two * (from_nat[Real](k.suc) * pi)).cos = x.cos
            p(k.suc)
        }
    }
    p(Nat.0) and forall(k: Nat) { p(k) implies p(k.suc) }
    Nat.induction(p)
    p(n)
}

// Further period and parity consequences.

/// The square of sine is unchanged by the two pi shift.
theorem sin_sq_period(x: Real) {
    (x + two * pi).sin.pow(Nat.2) = x.sin.pow(Nat.2)
} by {
    sin_add_two_pi_two(x)
    (x + two * pi).sin = x.sin
    (x + two * pi).sin.pow(Nat.2) = x.sin.pow(Nat.2)
}

/// The square of cosine is unchanged by the two pi shift.
theorem cos_sq_period(x: Real) {
    (x + two * pi).cos.pow(Nat.2) = x.cos.pow(Nat.2)
} by {
    cos_add_two_pi_two(x)
    (x + two * pi).cos = x.cos
    (x + two * pi).cos.pow(Nat.2) = x.cos.pow(Nat.2)
}

/// The square of sine plus the square of cosine is one.
theorem cos_sq_add_sin_sq(x: Real) {
    x.cos.pow(Nat.2) + x.sin.pow(Nat.2) = Real.1
} by {
    sin_sq_add_cos_sq(x)
    x.sin.pow(Nat.2) + x.cos.pow(Nat.2) = Real.1
    add_comm(x.sin.pow(Nat.2), x.cos.pow(Nat.2))
    x.sin.pow(Nat.2) + x.cos.pow(Nat.2) = x.cos.pow(Nat.2) + x.sin.pow(Nat.2)
    x.cos.pow(Nat.2) + x.sin.pow(Nat.2) = Real.1
}

/// The cosine of a difference in terms of the products.
theorem cos_add_neg(x: Real, y: Real) {
    (x - y).cos = x.cos * y.cos + x.sin * y.sin
} by {
    cos_add(x, -y)
    (x + (-y)).cos = x.cos * (-y).cos - x.sin * (-y).sin
    x + (-y) = x - y
    (x - y).cos = x.cos * (-y).cos - x.sin * (-y).sin
    cos_neg(y)
    (-y).cos = y.cos
    sin_neg(y)
    (-y).sin = -y.sin
    (x - y).cos = x.cos * y.cos - x.sin * (-y.sin)
    x.cos * y.cos - x.sin * (-y.sin) = x.cos * y.cos + x.sin * y.sin
    (x - y).cos = x.cos * y.cos + x.sin * y.sin
}

/// Secant is periodic with period two pi.
theorem sec_add_two_pi(x: Real) {
    sec(x + two * pi) = sec(x)
} by {
    sec(x + two * pi) = Real.1 / (x + two * pi).cos
    cos_add_two_pi_two(x)
    (x + two * pi).cos = x.cos
    Real.1 / (x + two * pi).cos = Real.1 / x.cos
    sec(x) = Real.1 / x.cos
    sec(x + two * pi) = sec(x)
}

/// Cosecant is periodic with period two pi.
theorem csc_add_two_pi(x: Real) {
    csc(x + two * pi) = csc(x)
} by {
    csc(x + two * pi) = Real.1 / (x + two * pi).sin
    sin_add_two_pi_two(x)
    (x + two * pi).sin = x.sin
    Real.1 / (x + two * pi).sin = Real.1 / x.sin
    csc(x) = Real.1 / x.sin
    csc(x + two * pi) = csc(x)
}

/// Tangent is periodic with period two pi.
theorem tan_add_two_pi(x: Real) {
    tan(x + two * pi) = tan(x)
} by {
    tan(x + two * pi) = (x + two * pi).sin / (x + two * pi).cos
    sin_add_two_pi_two(x)
    (x + two * pi).sin = x.sin
    cos_add_two_pi_two(x)
    (x + two * pi).cos = x.cos
    (x + two * pi).sin / (x + two * pi).cos = x.sin / x.cos
    tan(x) = x.sin / x.cos
    tan(x + two * pi) = tan(x)
}

/// Cotangent is periodic with period two pi.
theorem cot_add_two_pi(x: Real) {
    cot(x + two * pi) = cot(x)
} by {
    cot(x + two * pi) = (x + two * pi).cos / (x + two * pi).sin
    sin_add_two_pi_two(x)
    (x + two * pi).sin = x.sin
    cos_add_two_pi_two(x)
    (x + two * pi).cos = x.cos
    (x + two * pi).cos / (x + two * pi).sin = x.cos / x.sin
    cot(x) = x.cos / x.sin
    cot(x + two * pi) = cot(x)
}

// Derivative corollaries of the trig identities.
//
// These use the chain rule and the quotient rule from the calculus API to
// differentiate scaled and composed trig functions.

/// The function x ↦ 2 x.sin has derivative 2 x.cos.
theorem sin_scaled_derivative(x: Real) {
    has_derivative_at(compose(affine_real(two, Real.0), Real.sin), x, two * x.cos)
} by {
    sin_has_derivative_at(x)
    has_derivative_at(Real.sin, x, x.cos)
    derivative_affine_real_after(Real.sin, two, Real.0, x, x.cos)
    has_derivative_at(compose(affine_real(two, Real.0), Real.sin), x, two * x.cos)
}

/// The function x ↦ 2 x.cos has derivative -2 x.sin.
theorem cos_scaled_derivative(x: Real) {
    has_derivative_at(compose(affine_real(two, Real.0), Real.cos), x, two * (-x.sin))
} by {
    cos_has_derivative_at(x)
    has_derivative_at(Real.cos, x, -x.sin)
    derivative_affine_real_after(Real.cos, two, Real.0, x, -x.sin)
    has_derivative_at(compose(affine_real(two, Real.0), Real.cos), x, two * (-x.sin))
}

/// The function x ↦ (2x).sin has derivative 2 (2x).cos.
theorem sin_two_derivative(x: Real) {
    has_derivative_at(compose(Real.sin, affine_real(two, Real.0)), x, (two * x).cos * two)
} by {
    affine_real_has_derivative_at(two, Real.0, x)
    has_derivative_at(affine_real(two, Real.0), x, two)
    affine_real(two, Real.0, x) = two * x + Real.0
    two * x + Real.0 = two * x
    affine_real(two, Real.0, x) = two * x
    sin_has_derivative_at(affine_real(two, Real.0, x))
    has_derivative_at(Real.sin, affine_real(two, Real.0, x), (affine_real(two, Real.0, x)).cos)
    derivative_compose(affine_real(two, Real.0), Real.sin, x, two, (affine_real(two, Real.0, x)).cos)
    has_derivative_at(compose(Real.sin, affine_real(two, Real.0)), x, (affine_real(two, Real.0, x)).cos * two)
    (affine_real(two, Real.0, x)).cos = (two * x).cos
    (affine_real(two, Real.0, x)).cos * two = (two * x).cos * two
    has_derivative_at(compose(Real.sin, affine_real(two, Real.0)), x, (two * x).cos * two)
}

/// The function x ↦ (2x).cos has derivative -2 (2x).sin.
theorem cos_two_derivative(x: Real) {
    has_derivative_at(compose(Real.cos, affine_real(two, Real.0)), x, (-(two * x).sin) * two)
} by {
    affine_real_has_derivative_at(two, Real.0, x)
    has_derivative_at(affine_real(two, Real.0), x, two)
    affine_real(two, Real.0, x) = two * x + Real.0
    two * x + Real.0 = two * x
    affine_real(two, Real.0, x) = two * x
    cos_has_derivative_at(affine_real(two, Real.0, x))
    has_derivative_at(Real.cos, affine_real(two, Real.0, x), -(affine_real(two, Real.0, x)).sin)
    derivative_compose(affine_real(two, Real.0), Real.cos, x, two, -(affine_real(two, Real.0, x)).sin)
    has_derivative_at(compose(Real.cos, affine_real(two, Real.0)), x, (-(affine_real(two, Real.0, x)).sin) * two)
    (affine_real(two, Real.0, x)).sin = (two * x).sin
    -(affine_real(two, Real.0, x)).sin = -(two * x).sin
    (-(affine_real(two, Real.0, x)).sin) * two = (-(two * x).sin) * two
    has_derivative_at(compose(Real.cos, affine_real(two, Real.0)), x, (-(two * x).sin) * two)
}

/// The square of secant is the reciprocal of the square of cosine.
theorem sec_sq_eq(x: Real) {
    x.cos != Real.0 implies sec(x).pow(Nat.2) = Real.1 / x.cos.pow(Nat.2)
} by {
    if x.cos != Real.0 {
        sec(x) = Real.1 / x.cos
        sec(x).pow(Nat.2) = (Real.1 / x.cos).pow(Nat.2)
        pow_suc(Real.1 / x.cos, Nat.1)
        (Real.1 / x.cos).pow(Nat.2) = (Real.1 / x.cos) * (Real.1 / x.cos)
        mul_div(Real.1, x.cos, Real.1, x.cos)
        (Real.1 / x.cos) * (Real.1 / x.cos) = (Real.1 * Real.1) / (x.cos * x.cos)
        Real.1 * Real.1 = Real.1
        (Real.1 * Real.1) / (x.cos * x.cos) = Real.1 / (x.cos * x.cos)
        pow_suc(x.cos, Nat.1)
        x.cos.pow(Nat.2) = x.cos * x.cos
        Real.1 / (x.cos * x.cos) = Real.1 / x.cos.pow(Nat.2)
        sec(x).pow(Nat.2) = Real.1 / x.cos.pow(Nat.2)
    }
}

/// The tangent function equals the pointwise quotient of sine and cosine.
theorem tan_eq_pointwise_div {
    tan = pointwise_div_real(Real.sin, Real.cos)
} by {
    forall(y: Real) {
        tan(y) = y.sin / y.cos
        pointwise_div_real_apply(Real.sin, Real.cos, y)
        pointwise_div_real(Real.sin, Real.cos, y) = y.sin / y.cos
        tan(y) = pointwise_div_real(Real.sin, Real.cos, y)
    }
    function_extensionality(tan, pointwise_div_real(Real.sin, Real.cos))
    tan = pointwise_div_real(Real.sin, Real.cos)
}

/// The tangent has derivative secant squared wherever cosine is nonzero.
theorem tan_has_derivative_at(x: Real) {
    x.cos != Real.0 implies has_derivative_at(tan, x, sec(x).pow(Nat.2))
} by {
    if x.cos != Real.0 {
        define derivative_pred(h: Real -> Real) -> Bool {
            has_derivative_at(h, x, sec(x).pow(Nat.2))
        }
        sin_has_derivative_at(x)
        has_derivative_at(Real.sin, x, x.cos)
        cos_has_derivative_at(x)
        has_derivative_at(Real.cos, x, -x.sin)
        quotient_has_derivative_at(Real.sin, Real.cos, x, x.cos, -x.sin)
        has_derivative_at(pointwise_div_real(Real.sin, Real.cos), x,
            (x.cos * x.cos - x.sin * (-x.sin)) / (x.cos * x.cos))
        x.sin * (-x.sin) = -(x.sin * x.sin)
        x.cos * x.cos - x.sin * (-x.sin) = x.cos * x.cos + x.sin * x.sin
        pow_suc(x.cos, Nat.1)
        x.cos.pow(Nat.2) = x.cos * x.cos
        pow_suc(x.sin, Nat.1)
        x.sin.pow(Nat.2) = x.sin * x.sin
        sin_sq_add_cos_sq(x)
        x.sin.pow(Nat.2) + x.cos.pow(Nat.2) = Real.1
        x.cos.pow(Nat.2) + x.sin.pow(Nat.2) = Real.1
        x.cos * x.cos + x.sin * x.sin = x.cos.pow(Nat.2) + x.sin.pow(Nat.2)
        x.cos * x.cos - x.sin * (-x.sin) = Real.1
        (x.cos * x.cos - x.sin * (-x.sin)) / (x.cos * x.cos) = Real.1 / (x.cos * x.cos)
        Real.1 / (x.cos * x.cos) = Real.1 / x.cos.pow(Nat.2)
        sec_sq_eq(x)
        sec(x).pow(Nat.2) = Real.1 / x.cos.pow(Nat.2)
        Real.1 / x.cos.pow(Nat.2) = sec(x).pow(Nat.2)
        (x.cos * x.cos - x.sin * (-x.sin)) / (x.cos * x.cos) = sec(x).pow(Nat.2)
        has_derivative_at(pointwise_div_real(Real.sin, Real.cos), x, sec(x).pow(Nat.2))
        derivative_pred(pointwise_div_real(Real.sin, Real.cos))
        tan_eq_pointwise_div
        tan = pointwise_div_real(Real.sin, Real.cos)
        function_eq_transport_predicate_rev(derivative_pred, tan, pointwise_div_real(Real.sin, Real.cos))
        has_derivative_at(tan, x, sec(x).pow(Nat.2))
    }
}

// Further rearrangements and cofunction values.

/// The difference of the squares of sine and cosine is minus the cosine of
/// twice the angle.
theorem sin_sq_sub_cos_sq(x: Real) {
    x.sin.pow(Nat.2) - x.cos.pow(Nat.2) = -(two * x).cos
} by {
    cos_two(x)
    (two * x).cos = x.cos.pow(Nat.2) - x.sin.pow(Nat.2)
    x.sin.pow(Nat.2) - x.cos.pow(Nat.2) = -(x.cos.pow(Nat.2) - x.sin.pow(Nat.2))
    -(x.cos.pow(Nat.2) - x.sin.pow(Nat.2)) = -(two * x).cos
    x.sin.pow(Nat.2) - x.cos.pow(Nat.2) = -(two * x).cos
}

/// One minus the cosine of twice an angle is twice the square of sine.
theorem one_minus_cos_two(x: Real) {
    Real.1 - (two * x).cos = two * x.sin.pow(Nat.2)
} by {
    cos_two_alt2(x)
    (two * x).cos = Real.1 - two * x.sin.pow(Nat.2)
    Real.1 - (two * x).cos = Real.1 - (Real.1 - two * x.sin.pow(Nat.2))
    sub_minus_sub(Real.1, two * x.sin.pow(Nat.2))
    Real.1 - (Real.1 - two * x.sin.pow(Nat.2)) = two * x.sin.pow(Nat.2)
    Real.1 - (two * x).cos = two * x.sin.pow(Nat.2)
}

/// One plus the cosine of twice an angle is twice the square of cosine.
theorem one_add_cos_two(x: Real) {
    Real.1 + (two * x).cos = two * x.cos.pow(Nat.2)
} by {
    cos_two_alt1(x)
    (two * x).cos = two * x.cos.pow(Nat.2) - Real.1
    Real.1 + (two * x).cos = Real.1 + (two * x.cos.pow(Nat.2) - Real.1)
    Real.1 + (two * x.cos.pow(Nat.2) - Real.1) = Real.1 + two * x.cos.pow(Nat.2) - Real.1
    Real.1 + two * x.cos.pow(Nat.2) - Real.1 = two * x.cos.pow(Nat.2)
    Real.1 + (two * x.cos.pow(Nat.2) - Real.1) = two * x.cos.pow(Nat.2)
    Real.1 + (two * x).cos = two * x.cos.pow(Nat.2)
}

/// The sine of pi over two plus x is the cosine of x.
theorem sin_pi_over_two_add(x: Real) {
    (pi_over_two + x).sin = x.cos
} by {
    sin_add_pi_over_two(x)
    (x + pi_over_two).sin = x.cos
    add_comm(x, pi_over_two)
    x + pi_over_two = pi_over_two + x
    (x + pi_over_two).sin = (pi_over_two + x).sin
    (pi_over_two + x).sin = x.cos
}

/// The cosine of pi over two plus x is minus the sine of x.
theorem cos_pi_over_two_add(x: Real) {
    (pi_over_two + x).cos = -x.sin
} by {
    cos_add_pi_over_two(x)
    (x + pi_over_two).cos = -x.sin
    add_comm(x, pi_over_two)
    x + pi_over_two = pi_over_two + x
    (x + pi_over_two).cos = (pi_over_two + x).cos
    (pi_over_two + x).cos = -x.sin
}

/// The cosine of x minus pi over two is the sine of x.
theorem cos_x_sub_pi_over_two(x: Real) {
    (x - pi_over_two).cos = x.sin
} by {
    cos_add_neg(x, pi_over_two)
    (x - pi_over_two).cos = x.cos * pi_over_two.cos + x.sin * pi_over_two.sin
    cos_pi_over_two_zero
    pi_over_two.cos = Real.0
    sin_pi_over_two_one
    pi_over_two.sin = Real.1
    x.cos * pi_over_two.cos + x.sin * pi_over_two.sin = x.cos * Real.0 + x.sin * Real.1
    x.cos * Real.0 = Real.0
    x.sin * Real.1 = x.sin
    x.cos * Real.0 + x.sin * Real.1 = Real.0 + x.sin
    Real.0 + x.sin = x.sin
    (x - pi_over_two).cos = x.sin
}

/// The sine of x minus pi over two is minus the cosine of x.
theorem sin_x_sub_pi_over_two(x: Real) {
    (x - pi_over_two).sin = -x.cos
} by {
    sin_add(x, -pi_over_two)
    (x + (-pi_over_two)).sin = x.sin * (-pi_over_two).cos + x.cos * (-pi_over_two).sin
    x + (-pi_over_two) = x - pi_over_two
    (x - pi_over_two).sin = x.sin * (-pi_over_two).cos + x.cos * (-pi_over_two).sin
    cos_neg(pi_over_two)
    (-pi_over_two).cos = pi_over_two.cos
    sin_neg(pi_over_two)
    (-pi_over_two).sin = -pi_over_two.sin
    cos_pi_over_two_zero
    pi_over_two.cos = Real.0
    sin_pi_over_two_one
    pi_over_two.sin = Real.1
    x.sin * (-pi_over_two).cos + x.cos * (-pi_over_two).sin = x.sin * Real.0 + x.cos * -Real.1
    x.sin * Real.0 = Real.0
    x.cos * -Real.1 = -x.cos
    x.sin * Real.0 + x.cos * -Real.1 = Real.0 + -x.cos
    Real.0 + -x.cos = -x.cos
    (x - pi_over_two).sin = -x.cos
}

/// The sine of pi plus x is minus the sine of x.
theorem sin_pi_add(x: Real) {
    (pi + x).sin = -x.sin
} by {
    sin_add_pi(x)
    (x + pi).sin = -x.sin
    add_comm(x, pi)
    x + pi = pi + x
    (x + pi).sin = (pi + x).sin
    (pi + x).sin = -x.sin
}

/// The cosine of pi plus x is minus the cosine of x.
theorem cos_pi_add(x: Real) {
    (pi + x).cos = -x.cos
} by {
    cos_add_pi(x)
    (x + pi).cos = -x.cos
    add_comm(x, pi)
    x + pi = pi + x
    (x + pi).cos = (pi + x).cos
    (pi + x).cos = -x.cos
}

/// The sine of two pi plus x is the sine of x.
theorem sin_two_pi_add(x: Real) {
    (two * pi + x).sin = x.sin
} by {
    sin_add_two_pi_two(x)
    (x + two * pi).sin = x.sin
    add_comm(x, two * pi)
    x + two * pi = two * pi + x
    (x + two * pi).sin = (two * pi + x).sin
    (two * pi + x).sin = x.sin
}

/// The cosine of two pi plus x is the cosine of x.
theorem cos_two_pi_add(x: Real) {
    (two * pi + x).cos = x.cos
} by {
    cos_add_two_pi_two(x)
    (x + two * pi).cos = x.cos
    add_comm(x, two * pi)
    x + two * pi = two * pi + x
    (x + two * pi).cos = (two * pi + x).cos
    (two * pi + x).cos = x.cos
}

/// The square of secant minus one is the square of tangent.
theorem sec_sq_sub_one(x: Real) {
    x.cos != Real.0 implies sec(x).pow(Nat.2) - Real.1 = tan(x).pow(Nat.2)
} by {
    if x.cos != Real.0 {
        sec_sq_sub_tan_sq(x)
        sec(x).pow(Nat.2) - tan(x).pow(Nat.2) = Real.1
        sec(x).pow(Nat.2) - tan(x).pow(Nat.2) - Real.1 = Real.0
        sec(x).pow(Nat.2) - tan(x).pow(Nat.2) - Real.1 = sec(x).pow(Nat.2) - (tan(x).pow(Nat.2) + Real.1)
        sec(x).pow(Nat.2) - (tan(x).pow(Nat.2) + Real.1) = Real.0
        sec(x).pow(Nat.2) = tan(x).pow(Nat.2) + Real.1
        sec(x).pow(Nat.2) - Real.1 = tan(x).pow(Nat.2)
    }
}

/// The tangent of two pi is zero.
theorem tan_two_pi {
    tan(pi + pi) = Real.0
} by {
    tan(pi + pi) = (pi + pi).sin / (pi + pi).cos
    sin_two_pi_zero
    (pi + pi).sin = Real.0
    cos_two_pi_one
    (pi + pi).cos = Real.1
    (pi + pi).sin / (pi + pi).cos = Real.0 / Real.1
    Real.0 / Real.1 = Real.0
    tan(pi + pi) = Real.0
}

/// The secant of zero is one.
theorem sec_zero {
    sec(Real.0) = Real.1
} by {
    sec(Real.0) = Real.1 / (Real.0).cos
    cos_zero
    (Real.0).cos = Real.1
    Real.1 / (Real.0).cos = Real.1 / Real.1
    Real.1 / Real.1 = Real.1
    sec(Real.0) = Real.1
}

/// The cotangent of pi over two is zero.
theorem cot_pi_over_two {
    cot(pi_over_two) = Real.0
} by {
    cot(pi_over_two) = pi_over_two.cos / pi_over_two.sin
    cos_pi_over_two_zero
    pi_over_two.cos = Real.0
    sin_pi_over_two_one
    pi_over_two.sin = Real.1
    pi_over_two.cos / pi_over_two.sin = Real.0 / Real.1
    Real.0 / Real.1 = Real.0
    cot(pi_over_two) = Real.0
}

/// The cosecant of pi over two is one.
theorem csc_pi_over_two {
    csc(pi_over_two) = Real.1
} by {
    csc(pi_over_two) = Real.1 / pi_over_two.sin
    sin_pi_over_two_one
    pi_over_two.sin = Real.1
    Real.1 / pi_over_two.sin = Real.1 / Real.1
    Real.1 / Real.1 = Real.1
    csc(pi_over_two) = Real.1
}

/// Twice the product of sine and cosine of the same angle is the sine of
/// twice the angle.
theorem sin_mul_cos_two(x: Real) {
    x.sin * x.cos + x.sin * x.cos = (two * x).sin
} by {
    sin_two(x)
    (two * x).sin = two * x.sin * x.cos
    x.sin * x.cos + x.sin * x.cos = two * (x.sin * x.cos)
    two * (x.sin * x.cos) = two * x.sin * x.cos
    x.sin * x.cos + x.sin * x.cos = two * x.sin * x.cos
    x.sin * x.cos + x.sin * x.cos = (two * x).sin
}

// Parity of the reciprocal functions and final shifts.

/// The inverse of a negation is the negation of the inverse.
theorem neg_inverse_eq(a: Real) {
    a != Real.0 implies (-a).inverse = -a.inverse
} by {
    if a != Real.0 {
        neg_mul_neg(a, a.inverse)
        (-a) * (-(a.inverse)) = a * a.inverse
        mul_inverse(a)
        a * a.inverse = Real.1
        (-a) * (-(a.inverse)) = Real.1
        unique_inverse[Real](-a, -(a.inverse))
        -(a.inverse) = (-a).inverse
        (-a).inverse = -a.inverse
    }
}

/// Secant is even: sec(-x) = sec(x).
theorem sec_neg(x: Real) {
    sec(-x) = sec(x)
} by {
    sec(-x) = Real.1 / (-x).cos
    cos_neg(x)
    (-x).cos = x.cos
    Real.1 / (-x).cos = Real.1 / x.cos
    sec(x) = Real.1 / x.cos
    sec(-x) = sec(x)
}

/// Cosecant is odd: csc(-x) = -csc(x) wherever sine is nonzero.
theorem csc_neg(x: Real) {
    x.sin != Real.0 implies csc(-x) = -csc(x)
} by {
    if x.sin != Real.0 {
        csc(-x) = Real.1 / (-x).sin
        sin_neg(x)
        (-x).sin = -x.sin
        Real.1 / (-x).sin = Real.1 / (-x.sin)
        Real.1 / (-x.sin) = Real.1 * (-x.sin).inverse
        neg_inverse_eq(x.sin)
        (-x.sin).inverse = -x.sin.inverse
        Real.1 * (-x.sin).inverse = Real.1 * (-x.sin.inverse)
        Real.1 * (-x.sin.inverse) = -x.sin.inverse
        Real.1 / (-x.sin) = -x.sin.inverse
        csc(x) = Real.1 / x.sin
        Real.1 / x.sin = Real.1 * x.sin.inverse
        Real.1 * x.sin.inverse = x.sin.inverse
        -(Real.1 / x.sin) = -x.sin.inverse
        -csc(x) = -(Real.1 / x.sin)
        Real.1 / (-x.sin) = -csc(x)
        csc(-x) = -csc(x)
    }
}

/// Cotangent is odd: cot(-x) = -cot(x) wherever sine and cosine are nonzero.
theorem cot_neg(x: Real) {
    x.sin != Real.0 and x.cos != Real.0 implies cot(-x) = -cot(x)
} by {
    if x.sin != Real.0 and x.cos != Real.0 {
        cot(-x) = (-x).cos / (-x).sin
        cos_neg(x)
        (-x).cos = x.cos
        sin_neg(x)
        (-x).sin = -x.sin
        (-x).cos / (-x).sin = x.cos / (-x.sin)
        x.cos / (-x.sin) = x.cos * (-x.sin).inverse
        neg_inverse_eq(x.sin)
        (-x.sin).inverse = -x.sin.inverse
        x.cos * (-x.sin).inverse = x.cos * (-x.sin.inverse)
        mul_neg_right(x.cos, x.sin.inverse)
        x.cos * (-x.sin.inverse) = -(x.cos * x.sin.inverse)
        x.cos * x.sin.inverse = x.cos / x.sin
        -(x.cos * x.sin.inverse) = -(x.cos / x.sin)
        x.cos / (-x.sin) = -(x.cos / x.sin)
        cot(x) = x.cos / x.sin
        -(x.cos / x.sin) = -cot(x)
        x.cos / (-x.sin) = -cot(x)
        cot(-x) = -cot(x)
    }
}

/// The square of sine in terms of cosine squared.
theorem sin_sq_alt(x: Real) {
    x.sin.pow(Nat.2) = Real.1 - x.cos.pow(Nat.2)
} by {
    sin_sq_add_cos_sq(x)
    x.sin.pow(Nat.2) + x.cos.pow(Nat.2) = Real.1
    x.sin.pow(Nat.2) = Real.1 - x.cos.pow(Nat.2)
}

/// The square of cosine in terms of sine squared.
theorem cos_sq_alt(x: Real) {
    x.cos.pow(Nat.2) = Real.1 - x.sin.pow(Nat.2)
} by {
    sin_sq_add_cos_sq(x)
    x.sin.pow(Nat.2) + x.cos.pow(Nat.2) = Real.1
    x.cos.pow(Nat.2) = Real.1 - x.sin.pow(Nat.2)
}

/// The square of sine is unchanged by the pi shift.
theorem sin_sq_period_pi(x: Real) {
    (x + pi).sin.pow(Nat.2) = x.sin.pow(Nat.2)
} by {
    sin_add_pi(x)
    (x + pi).sin = -x.sin
    (x + pi).sin.pow(Nat.2) = (-x.sin).pow(Nat.2)
    pow_suc(-x.sin, Nat.1)
    (-x.sin).pow(Nat.2) = (-x.sin) * (-x.sin)
    neg_mul_neg(x.sin, x.sin)
    (-x.sin) * (-x.sin) = x.sin * x.sin
    pow_suc(x.sin, Nat.1)
    x.sin.pow(Nat.2) = x.sin * x.sin
    (-x.sin).pow(Nat.2) = x.sin.pow(Nat.2)
    (x + pi).sin.pow(Nat.2) = x.sin.pow(Nat.2)
}

/// The square of cosine is unchanged by the pi shift.
theorem cos_sq_period_pi(x: Real) {
    (x + pi).cos.pow(Nat.2) = x.cos.pow(Nat.2)
} by {
    cos_add_pi(x)
    (x + pi).cos = -x.cos
    (x + pi).cos.pow(Nat.2) = (-x.cos).pow(Nat.2)
    pow_suc(-x.cos, Nat.1)
    (-x.cos).pow(Nat.2) = (-x.cos) * (-x.cos)
    neg_mul_neg(x.cos, x.cos)
    (-x.cos) * (-x.cos) = x.cos * x.cos
    pow_suc(x.cos, Nat.1)
    x.cos.pow(Nat.2) = x.cos * x.cos
    (-x.cos).pow(Nat.2) = x.cos.pow(Nat.2)
    (x + pi).cos.pow(Nat.2) = x.cos.pow(Nat.2)
}

/// The cosine of x minus pi is minus the cosine of x.
theorem cos_sub_pi(x: Real) {
    (x - pi).cos = -x.cos
} by {
    cos_add_neg(x, pi)
    (x - pi).cos = x.cos * pi.cos + x.sin * pi.sin
    cos_pi_neg_one
    pi.cos = -Real.1
    sin_pi_zero
    pi.sin = Real.0
    x.cos * pi.cos + x.sin * pi.sin = x.cos * -Real.1 + x.sin * Real.0
    x.cos * -Real.1 = -x.cos
    x.sin * Real.0 = Real.0
    x.cos * -Real.1 + x.sin * Real.0 = -x.cos + Real.0
    -x.cos + Real.0 = -x.cos
    (x - pi).cos = -x.cos
}

/// The sine of x minus pi is minus the sine of x.
theorem sin_sub_pi(x: Real) {
    (x - pi).sin = -x.sin
} by {
    sin_add(x, -pi)
    (x + (-pi)).sin = x.sin * (-pi).cos + x.cos * (-pi).sin
    x + (-pi) = x - pi
    (x - pi).sin = x.sin * (-pi).cos + x.cos * (-pi).sin
    cos_neg(pi)
    (-pi).cos = pi.cos
    sin_neg(pi)
    (-pi).sin = -pi.sin
    cos_pi_neg_one
    pi.cos = -Real.1
    sin_pi_zero
    pi.sin = Real.0
    x.sin * (-pi).cos + x.cos * (-pi).sin = x.sin * -Real.1 + x.cos * -Real.0
    x.sin * -Real.1 = -x.sin
    x.cos * -Real.0 = Real.0
    x.sin * -Real.1 + x.cos * -Real.0 = -x.sin + Real.0
    -x.sin + Real.0 = -x.sin
    (x - pi).sin = -x.sin
}
