from data.basic.functions import compose
from nat import Nat
from list import partial, sum, map, List
from list import reverse, reverse_get_idx
from list import map_range, list_extensionality, scalar_mul
from real.real_ring import Real
from real.real_field import mul_inverse_comm
from algebra.semigroup import mul_fn
from algebra.add_semigroup import add_fn

numerals Real

// This file defines the Cauchy product of two series and proves theorems about it.

attributes Real {
    // Placeholder to let other modules import Real from here.
}

/// The coefficient function for the Cauchy product at index n.
/// For a fixed n, this returns a function mapping k to a(k) * b(n-k).
define cauchy_coefficient(a: Nat -> Real, b: Nat -> Real, n: Nat) -> (Nat -> Real) {
    function(k: Nat) { a(k) * b(n - k) }
}

/// The Cauchy product of two sequences at index n.
/// This computes ∑_{k=0}^{n} a(k) * b(n-k).
define cauchy_product(a: Nat -> Real, b: Nat -> Real, n: Nat) -> Real {
    sum(map(n.suc.range, cauchy_coefficient(a, b, n)))
}

/// The sequence of Cauchy products.
define cauchy_seq(a: Nat -> Real, b: Nat -> Real) -> (Nat -> Real) {
    function(n: Nat) { cauchy_product(a, b, n) }
}

theorem cauchy_product_zero(a: Nat -> Real, b: Nat -> Real) {
    cauchy_product(a, b, Nat.0) = a(Nat.0) * b(Nat.0)
} by {
    let mapped = map(Nat.1.range, function(k: Nat) { a(k) * b(Nat.0 - k) })
    sum(map(Nat.0.suc.range, cauchy_coefficient(a, b, Nat.0))) = cauchy_product(a, b, Nat.0)
    a(Nat.0) * b(Nat.0 - Nat.0) = cauchy_coefficient(a, b, Nat.0, Nat.0)
    sum(map(Nat.0.suc.range, cauchy_coefficient(a, b, Nat.0))) = partial(cauchy_coefficient(a, b, Nat.0), Nat.0.suc)
    partial(cauchy_coefficient(a, b, Nat.0), Nat.1) = cauchy_coefficient(a, b, Nat.0, Nat.0)
    Nat.0 - Nat.0 = Nat.0
}

/// If we map a list with a function that always returns zero, the sum is zero.
theorem sum_map_zero[T](items: List[T], f: T -> Real) {
    (forall(x: T) { f(x) = Real.0 })
    implies
    sum(map(items, f)) = Real.0
} by {
    define p(xs: List[T]) -> Bool {
        (forall(x: T) { f(x) = Real.0 })
        implies
        sum(map(xs, f)) = Real.0
    }

    // Base case: empty list
    map[T, Real](List.nil[T], f) = List.nil[Real]
    sum[Real](map[T, Real](List.nil[T], f)) = Real.0
    p(List.nil)

    // Inductive step
    forall(head: T, tail: List[T]) {
        if p(tail) {
            if forall(x: T) { f(x) = Real.0 } {
                p(tail)
                sum(map(tail, f)) = Real.0
                f(head) = Real.0
                Real.0 + Real.0 = Real.0
                f(head) + sum(map(tail, f)) = Real.0
                f(head) + sum(map(tail, f)) = sum(List.cons(f(head), map(tail, f)))
                List.cons(f(head), map(tail, f)) = map(List.cons(head, tail), f)
                sum(map(List.cons(head, tail), f)) = Real.0
                p(List.cons(head, tail))
            }
            if forall(x: T) { f(x) = Real.0 } {
                p(tail)
                sum(map(tail, f)) = Real.0
                f(head) = Real.0
                Real.0 + Real.0 = Real.0
                f(head) + sum(map(tail, f)) = Real.0
                f(head) + sum(map(tail, f)) = sum(List.cons(f(head), map(tail, f)))
                List.cons(f(head), map(tail, f)) = map(List.cons(head, tail), f)
                sum(map(List.cons(head, tail), f)) = Real.0
            }
            (forall(x: T) { f(x) = Real.0 }) implies sum(map(List.cons(head, tail), f)) = Real.0
            p(List.cons(head, tail))
        }
    }

    p(items)
}

/// If a is the constant zero function, then cauchy_coefficient returns constant zero.
theorem cauchy_coefficient_zero_left(b: Nat -> Real, n: Nat, k: Nat) {
    cauchy_coefficient(constant[Nat, Real](Real.0), b, n)(k) = Real.0
} by {
    let a = constant[Nat, Real](Real.0)
}

/// If a is constant zero, then cauchy_coefficient is the constant zero function.
theorem cauchy_coefficient_zero_left_fn(b: Nat -> Real, n: Nat) {
    cauchy_coefficient(constant[Nat, Real](Real.0), b, n) = constant[Nat, Real](Real.0)
} by {
    let a = constant[Nat, Real](Real.0)
    forall(k: Nat) {
        constant[Nat, Real](Real.0, k) = Real.0
        cauchy_coefficient(a, b, n, k) = constant[Nat, Real](Real.0, k)
    }
}

/// If b is the constant zero function, then cauchy_coefficient returns constant zero.
theorem cauchy_coefficient_zero_right(a: Nat -> Real, n: Nat, k: Nat) {
    cauchy_coefficient(a, constant[Nat, Real](Real.0), n)(k) = Real.0
} by {
    let b = constant[Nat, Real](Real.0)
}

/// If b is constant zero, then cauchy_coefficient is the constant zero function.
theorem cauchy_coefficient_zero_right_fn(a: Nat -> Real, n: Nat) {
    cauchy_coefficient(a, constant[Nat, Real](Real.0), n) = constant[Nat, Real](Real.0)
} by {
    let b = constant[Nat, Real](Real.0)
    forall(k: Nat) {
        constant[Nat, Real](Real.0, k) = Real.0
        cauchy_coefficient(a, b, n, k) = constant[Nat, Real](Real.0, k)
    }
}

/// Helper lemma: reverse of mapped range with cauchy_coefficient equals mapped range with swapped coefficients.
theorem cauchy_reverse_map_eq(a: Nat -> Real, b: Nat -> Real, n: Nat) {
    reverse(map(n.suc.range, cauchy_coefficient(a, b, n))) =
    map(n.suc.range, cauchy_coefficient(b, a, n))
} by {
    let left_list = reverse(map(n.suc.range, cauchy_coefficient(a, b, n)))
    let right_list = map(n.suc.range, cauchy_coefficient(b, a, n))

    map[Nat, Real](n.suc.range, cauchy_coefficient(a, b, n)).length = n.suc.range.length
    map[Nat, Real](n.suc.range, cauchy_coefficient(b, a, n)).length = n.suc.range.length
    reverse[Real](left_list).length = left_list.length
    reverse[Real](reverse[Real](map[Nat, Real](n.suc.range, cauchy_coefficient(a, b, n)))) = map[Nat, Real](n.suc.range, cauchy_coefficient(a, b, n))
    left_list.length = right_list.length

    forall(idx: Nat) {
        if idx < left_list.length {
            idx < n.suc

            let mapped = map(n.suc.range, cauchy_coefficient(a, b, n))
            n - idx <= n

            mapped.get_idx(n - idx) = Option.some(cauchy_coefficient(a, b, n)(n - idx))

            n - (n - idx) = idx

            a(n - idx) * b(n - (n - idx)) = cauchy_coefficient(a, b, n, n - idx)
            b(idx) * a(n - idx) = cauchy_coefficient(b, a, n, idx)
            b(n - (n - idx)) * a(n - idx) = a(n - idx) * b(n - (n - idx))
            map(n.suc.range, cauchy_coefficient(b, a, n)).length = n.suc.range.length
            reverse(mapped).length = mapped.length
            n.suc - Nat.1 = n
            n.suc.range.length = n.suc

            // Connect left_list to mapped via reverse
            left_list = reverse(mapped)
            left_list.get_idx(idx) = reverse(mapped).get_idx(idx)

            // reverse_get_idx gives us: reverse(mapped).get_idx(idx) = mapped.get_idx(mapped.length - 1 - idx)
            mapped.length = n.suc
            mapped.length - Nat.1 - idx = n - idx
            reverse(mapped).get_idx(idx) = mapped.get_idx(n - idx)

            // From map_range we know mapped.get_idx(n - idx)
            left_list.get_idx(idx) = Option.some(cauchy_coefficient(a, b, n)(n - idx))

            // Connect right_list to cauchy_coefficient(b, a, n)
            right_list.get_idx(idx) = Option.some(cauchy_coefficient(b, a, n)(idx))

            // Show coefficients are equal via commutativity
            cauchy_coefficient(a, b, n)(n - idx) = a(n - idx) * b(n - (n - idx))
            a(n - idx) * b(idx) = b(idx) * a(n - idx)
            cauchy_coefficient(b, a, n)(idx) = b(idx) * a(n - idx)

            // Final connection
            cauchy_coefficient(a, b, n)(n - idx) = cauchy_coefficient(b, a, n)(idx)

            left_list.get_idx(idx) = right_list.get_idx(idx)
        }
    }

    left_list.length = right_list.length and (forall(idx: Nat) { idx < left_list.length implies left_list.get_idx(idx) = right_list.get_idx(idx) })
    left_list = right_list
    reverse(map(n.suc.range, cauchy_coefficient(a, b, n))) = left_list
    map(n.suc.range, cauchy_coefficient(b, a, n)) = right_list
    reverse(map(n.suc.range, cauchy_coefficient(a, b, n))) = map(n.suc.range, cauchy_coefficient(b, a, n))
}

/// Cauchy product is commutative.
theorem cauchy_product_comm(a: Nat -> Real, b: Nat -> Real, n: Nat) {
    cauchy_product(a, b, n) = cauchy_product(b, a, n)
} by {
}

/// Cauchy product with a zero sequence on the left.
theorem cauchy_product_zero_left(b: Nat -> Real, n: Nat) {
    cauchy_product(constant[Nat, Real](Real.0), b, n) = Real.0
} by {
    from real.abs_conv import partial_constant_zero
    let a = constant[Nat, Real](Real.0)

    sum[Real](map[Nat, Real](n.suc.range, cauchy_coefficient(a, b, n))) = partial(cauchy_coefficient(a, b, n), n.suc)
    partial(cauchy_coefficient(a, b, n), n.suc) = partial(constant[Nat, Real](Real.0), n.suc)
    partial[Real](constant[Nat, Real](Real.0), n.suc) = Real.0
    forall(k: Nat) {
    }
    sum(map(n.suc.range, cauchy_coefficient(a, b, n))) = Real.0
}

/// Cauchy product with a zero sequence on the right.
theorem cauchy_product_zero_right(a: Nat -> Real, n: Nat) {
    cauchy_product(a, constant[Nat, Real](Real.0), n) = Real.0
} by {
    from real.abs_conv import partial_constant_zero
    let b = constant[Nat, Real](Real.0)

    sum[Real](map[Nat, Real](n.suc.range, cauchy_coefficient(a, b, n))) = partial(cauchy_coefficient(a, b, n), n.suc)
    partial(cauchy_coefficient(a, b, n), n.suc) = partial(constant[Nat, Real](Real.0), n.suc)
    partial[Real](constant[Nat, Real](Real.0), n.suc) = Real.0
}

/// Helper lemma: cauchy_coefficient distributes scalar multiplication in the first argument.
theorem cauchy_coefficient_scalar_left(c: Real, a: Nat -> Real, b: Nat -> Real, n: Nat, k: Nat) {
    cauchy_coefficient(mul_fn(c, a), b, n)(k) = c * cauchy_coefficient(a, b, n)(k)
}

/// Linearity in the first argument: scalar multiplication factors out.
theorem cauchy_product_scalar_left(c: Real, a: Nat -> Real, b: Nat -> Real, n: Nat) {
    cauchy_product(mul_fn(c, a), b, n) = c * cauchy_product(a, b, n)
} by {
    sum[Real](map[Nat, Real](n.suc.range, cauchy_coefficient(a, b, n))) = cauchy_product(a, b, n)
    sum[Real](map[Nat, Real](n.suc.range, cauchy_coefficient(mul_fn[Nat, Real](c, a), b, n))) = cauchy_product(mul_fn[Nat, Real](c, a), b, n)
    map[Nat, Real](n.suc.range, compose[Nat, Real, Real](scalar_mul(c), cauchy_coefficient(a, b, n))) = map[Real, Real](map[Nat, Real](n.suc.range, cauchy_coefficient(a, b, n)), scalar_mul(c))
    forall(k: Nat) {
        compose[Nat, Real, Real](scalar_mul(c), cauchy_coefficient(a, b, n), k) = scalar_mul(c, cauchy_coefficient(a, b, n, k))
        c * cauchy_coefficient(a, b, n, k) = scalar_mul(c, cauchy_coefficient(a, b, n, k))
        cauchy_coefficient(mul_fn(c, a), b, n, k) = compose[Nat, Real, Real](scalar_mul(c), cauchy_coefficient(a, b, n), k)
    }
    cauchy_coefficient(mul_fn(c, a), b, n) = compose(scalar_mul(c), cauchy_coefficient(a, b, n))

    sum(map(map(n.suc.range, cauchy_coefficient(a, b, n)), scalar_mul(c))) = c * sum(map(n.suc.range, cauchy_coefficient(a, b, n)))
}

/// Helper lemma: cauchy_coefficient distributes scalar multiplication in the second argument.
theorem cauchy_coefficient_scalar_right(c: Real, a: Nat -> Real, b: Nat -> Real, n: Nat, k: Nat) {
    cauchy_coefficient(a, mul_fn(c, b), n)(k) = c * cauchy_coefficient(a, b, n)(k)
}

/// Linearity in the second argument: scalar multiplication factors out.
theorem cauchy_product_scalar_right(c: Real, a: Nat -> Real, b: Nat -> Real, n: Nat) {
    cauchy_product(a, mul_fn(c, b), n) = c * cauchy_product(a, b, n)
} by {
    sum[Real](map[Nat, Real](n.suc.range, cauchy_coefficient(a, b, n))) = cauchy_product(a, b, n)
    sum[Real](map[Nat, Real](n.suc.range, cauchy_coefficient(a, mul_fn[Nat, Real](c, b), n))) = cauchy_product(a, mul_fn[Nat, Real](c, b), n)
    map[Nat, Real](n.suc.range, compose[Nat, Real, Real](scalar_mul(c), cauchy_coefficient(a, b, n))) = map[Real, Real](map[Nat, Real](n.suc.range, cauchy_coefficient(a, b, n)), scalar_mul(c))
    forall(k: Nat) {
        compose[Nat, Real, Real](scalar_mul(c), cauchy_coefficient(a, b, n), k) = scalar_mul(c, cauchy_coefficient(a, b, n, k))
        c * cauchy_coefficient(a, b, n, k) = scalar_mul(c, cauchy_coefficient(a, b, n, k))
        cauchy_coefficient(a, mul_fn(c, b), n, k) = compose[Nat, Real, Real](scalar_mul(c), cauchy_coefficient(a, b, n), k)
    }
    cauchy_coefficient(a, mul_fn(c, b), n) = compose(scalar_mul(c), cauchy_coefficient(a, b, n))

    sum(map(map(n.suc.range, cauchy_coefficient(a, b, n)), scalar_mul(c))) = c * sum(map(n.suc.range, cauchy_coefficient(a, b, n)))
}

/// Helper lemma: cauchy_coefficient distributes addition in the first argument.
theorem cauchy_coefficient_add_left(a: Nat -> Real, aa: Nat -> Real, b: Nat -> Real, n: Nat, k: Nat) {
    cauchy_coefficient(add_fn(a, aa), b, n)(k) = cauchy_coefficient(a, b, n)(k) + cauchy_coefficient(aa, b, n)(k)
} by {
    a(k) * b(n - k) + aa(k) * b(n - k) = cauchy_coefficient(a, b, n)(k) + cauchy_coefficient(aa, b, n)(k)
}

/// Distributivity in the first argument: addition distributes through the Cauchy product.
theorem cauchy_product_add_left(a: Nat -> Real, aa: Nat -> Real, b: Nat -> Real, n: Nat) {
    cauchy_product(add_fn(a, aa), b, n) = cauchy_product(a, b, n) + cauchy_product(aa, b, n)
} by {
    forall(k: Nat) {
        add_fn(cauchy_coefficient(a, b, n), cauchy_coefficient(aa, b, n), k) = cauchy_coefficient(a, b, n, k) + cauchy_coefficient(aa, b, n, k)
        cauchy_coefficient(add_fn(a, aa), b, n, k) = add_fn(cauchy_coefficient(a, b, n), cauchy_coefficient(aa, b, n), k)
    }
    cauchy_coefficient(add_fn(a, aa), b, n) = add_fn(cauchy_coefficient(a, b, n), cauchy_coefficient(aa, b, n))
    sum(map(n.suc.range, cauchy_coefficient(a, b, n))) + sum(map(n.suc.range, cauchy_coefficient(aa, b, n))) = sum(map(n.suc.range, add_fn(cauchy_coefficient(a, b, n), cauchy_coefficient(aa, b, n))))
    sum(map(n.suc.range, cauchy_coefficient(a, b, n))) = cauchy_product(a, b, n)
    sum(map(n.suc.range, cauchy_coefficient(aa, b, n))) = cauchy_product(aa, b, n)
    sum(map(n.suc.range, cauchy_coefficient(add_fn(a, aa), b, n))) = cauchy_product(add_fn(a, aa), b, n)
}

/// Helper lemma: cauchy_coefficient distributes addition in the second argument.
theorem cauchy_coefficient_add_right(a: Nat -> Real, b: Nat -> Real, bb: Nat -> Real, n: Nat, k: Nat) {
    cauchy_coefficient(a, add_fn(b, bb), n)(k) = cauchy_coefficient(a, b, n)(k) + cauchy_coefficient(a, bb, n)(k)
} by {
    a(k) * b(n - k) + a(k) * bb(n - k) = cauchy_coefficient(a, b, n)(k) + cauchy_coefficient(a, bb, n)(k)
}

/// Distributivity in the second argument: addition distributes through the Cauchy product.
theorem cauchy_product_add_right(a: Nat -> Real, b: Nat -> Real, bb: Nat -> Real, n: Nat) {
    cauchy_product(a, add_fn(b, bb), n) = cauchy_product(a, b, n) + cauchy_product(a, bb, n)
} by {
    forall(k: Nat) {
        add_fn(cauchy_coefficient(a, b, n), cauchy_coefficient(a, bb, n), k) = cauchy_coefficient(a, b, n, k) + cauchy_coefficient(a, bb, n, k)
        cauchy_coefficient(a, add_fn(b, bb), n, k) = add_fn(cauchy_coefficient(a, b, n), cauchy_coefficient(a, bb, n), k)
    }
    cauchy_coefficient(a, add_fn(b, bb), n) = add_fn(cauchy_coefficient(a, b, n), cauchy_coefficient(a, bb, n))
    sum(map(n.suc.range, cauchy_coefficient(a, b, n))) + sum(map(n.suc.range, cauchy_coefficient(a, bb, n))) = sum(map(n.suc.range, add_fn(cauchy_coefficient(a, b, n), cauchy_coefficient(a, bb, n))))
    sum(map(n.suc.range, cauchy_coefficient(a, b, n))) = cauchy_product(a, b, n)
    sum(map(n.suc.range, cauchy_coefficient(a, bb, n))) = cauchy_product(a, bb, n)
    sum(map(n.suc.range, cauchy_coefficient(a, add_fn(b, bb), n))) = cauchy_product(a, add_fn(b, bb), n)
}

/// Partial sum of Cauchy product with zero sequence on the left is zero.
theorem partial_cauchy_zero_left(b: Nat -> Real, n: Nat) {
    partial(cauchy_seq(constant[Nat, Real](Real.0), b), n) = Real.0
} by {
    from real.abs_conv import partial_constant_zero
    forall(m: Nat) {
        cauchy_seq(constant[Nat, Real](Real.0), b, m) = cauchy_product(constant[Nat, Real](Real.0), b, m)
        constant[Nat, Real](Real.0, m) = Real.0
        cauchy_seq(constant[Nat, Real](Real.0), b, m) = constant[Nat, Real](Real.0, m)
    }
    cauchy_seq(constant[Nat, Real](Real.0), b) = constant[Nat, Real](Real.0)
    forall(m: Nat) {
        cauchy_seq(constant[Nat, Real](Real.0), b, m) = cauchy_product(constant[Nat, Real](Real.0), b, m)
        constant[Nat, Real](Real.0, m) = Real.0
        cauchy_seq(constant[Nat, Real](Real.0), b, m) = constant[Nat, Real](Real.0, m)
    }
    cauchy_seq(constant[Nat, Real](Real.0), b) = constant[Nat, Real](Real.0)
    partial[Real](constant[Nat, Real](Real.0), n) = Real.0
}

/// Partial sum of Cauchy product with zero sequence on the right is zero.
theorem partial_cauchy_zero_right(a: Nat -> Real, n: Nat) {
    partial(cauchy_seq(a, constant[Nat, Real](Real.0)), n) = Real.0
} by {
    from real.abs_conv import partial_constant_zero
    forall(m: Nat) {
        cauchy_seq(a, constant[Nat, Real](Real.0), m) = cauchy_product(a, constant[Nat, Real](Real.0), m)
        constant[Nat, Real](Real.0, m) = Real.0
        cauchy_seq(a, constant[Nat, Real](Real.0), m) = constant[Nat, Real](Real.0, m)
    }
    cauchy_seq(a, constant[Nat, Real](Real.0)) = constant[Nat, Real](Real.0)
    forall(m: Nat) {
        cauchy_seq(a, constant[Nat, Real](Real.0), m) = cauchy_product(a, constant[Nat, Real](Real.0), m)
        constant[Nat, Real](Real.0, m) = Real.0
        cauchy_seq(a, constant[Nat, Real](Real.0), m) = constant[Nat, Real](Real.0, m)
    }
    cauchy_seq(a, constant[Nat, Real](Real.0)) = constant[Nat, Real](Real.0)
    partial[Real](constant[Nat, Real](Real.0), n) = Real.0
}

/// Partial sum distributes over addition in the first argument.
theorem partial_cauchy_add_left(a: Nat -> Real, aa: Nat -> Real, b: Nat -> Real, n: Nat) {
    partial(cauchy_seq(add_fn(a, aa), b), n) = partial(cauchy_seq(a, b), n) + partial(cauchy_seq(aa, b), n)
} by {
    // Show the sequences are pointwise equal
    forall(m: Nat) {
        cauchy_seq(add_fn(a, aa), b)(m) = cauchy_product(add_fn(a, aa), b, m)
        cauchy_seq(a, b)(m) = cauchy_product(a, b, m)
        cauchy_seq(aa, b)(m) = cauchy_product(aa, b, m)
        cauchy_product(a, b, m) + cauchy_product(aa, b, m) = cauchy_product(add_fn(a, aa), b, m)
        add_fn(cauchy_seq(a, b), cauchy_seq(aa, b))(m) = cauchy_seq(a, b)(m) + cauchy_seq(aa, b)(m)
        cauchy_seq(add_fn(a, aa), b)(m) = add_fn(cauchy_seq(a, b), cauchy_seq(aa, b))(m)
    }
    cauchy_seq(add_fn(a, aa), b) = add_fn(cauchy_seq(a, b), cauchy_seq(aa, b))
    partial(cauchy_seq(a, b), n) + partial(cauchy_seq(aa, b), n) = partial(add_fn(cauchy_seq(a, b), cauchy_seq(aa, b)), n)
}

/// Partial sum distributes over addition in the second argument.
theorem partial_cauchy_add_right(a: Nat -> Real, b: Nat -> Real, bb: Nat -> Real, n: Nat) {
    partial(cauchy_seq(a, add_fn(b, bb)), n) = partial(cauchy_seq(a, b), n) + partial(cauchy_seq(a, bb), n)
} by {
    forall(m: Nat) {
        cauchy_seq(a, add_fn(b, bb))(m) = cauchy_product(a, add_fn(b, bb), m)
        cauchy_seq(a, b)(m) = cauchy_product(a, b, m)
        cauchy_seq(a, bb)(m) = cauchy_product(a, bb, m)
        cauchy_product(a, b, m) + cauchy_product(a, bb, m) = cauchy_product(a, add_fn(b, bb), m)
        add_fn(cauchy_seq(a, b), cauchy_seq(a, bb))(m) = cauchy_seq(a, b)(m) + cauchy_seq(a, bb)(m)
        cauchy_seq(a, add_fn(b, bb))(m) = add_fn(cauchy_seq(a, b), cauchy_seq(a, bb))(m)
    }
    cauchy_seq(a, add_fn(b, bb)) = add_fn(cauchy_seq(a, b), cauchy_seq(a, bb))
    partial(cauchy_seq(a, b), n) + partial(cauchy_seq(a, bb), n) = partial(add_fn(cauchy_seq(a, b), cauchy_seq(a, bb)), n)
}

/// Scalar multiplication factors out of partial sum in the first argument.
theorem partial_cauchy_scalar_left(c: Real, a: Nat -> Real, b: Nat -> Real, n: Nat) {
    partial(cauchy_seq(mul_fn(c, a), b), n) = c * partial(cauchy_seq(a, b), n)
} by {
    forall(m: Nat) {
        cauchy_seq(mul_fn(c, a), b)(m) = cauchy_product(mul_fn(c, a), b, m)
        cauchy_seq(a, b)(m) = cauchy_product(a, b, m)
        c * cauchy_product(a, b, m) = cauchy_product(mul_fn(c, a), b, m)
        mul_fn(c, cauchy_seq(a, b))(m) = c * cauchy_seq(a, b)(m)
        cauchy_seq(mul_fn(c, a), b)(m) = mul_fn(c, cauchy_seq(a, b))(m)
    }
    cauchy_seq(mul_fn(c, a), b) = mul_fn(c, cauchy_seq(a, b))
    c * partial(cauchy_seq(a, b), n) = partial(mul_fn(c, cauchy_seq(a, b)), n)
}

/// Scalar multiplication factors out of partial sum in the second argument.
theorem partial_cauchy_scalar_right(c: Real, a: Nat -> Real, b: Nat -> Real, n: Nat) {
    partial(cauchy_seq(a, mul_fn(c, b)), n) = c * partial(cauchy_seq(a, b), n)
} by {
    forall(m: Nat) {
        cauchy_seq(a, mul_fn(c, b))(m) = cauchy_product(a, mul_fn(c, b), m)
        cauchy_seq(a, b)(m) = cauchy_product(a, b, m)
        c * cauchy_product(a, b, m) = cauchy_product(a, mul_fn(c, b), m)
        cauchy_seq(mul_fn(c, a), b)(m) = cauchy_product(mul_fn(c, a), b, m)
        c * cauchy_product(a, b, m) = cauchy_product(mul_fn(c, a), b, m)
        cauchy_seq(a, mul_fn(c, b))(m) = cauchy_seq(mul_fn(c, a), b)(m)
    }
    cauchy_seq(a, mul_fn(c, b)) = cauchy_seq(mul_fn(c, a), b)
    c * partial(cauchy_seq(a, b), n) = partial(cauchy_seq(mul_fn(c, a), b), n)
}

// Import additional definitions for the Cauchy product theorem
from real.rectangular_sum import rectangular_sum, nonneg_fn_2, prod_fn, product_sum, product_sum_factors, rectangular_sum_increasing_rows, rectangular_sum_increasing_cols
from real.triangular_sum import triangular_sum, diagonal, diagonal_sum, triangular_sum_step, triangular_sum_decomposition_parts
from real.double_sum import double_sum, double_sum_converges, abs_fn_2, rectangular_sum_decomposition, sub_fn_2, double_image_set, double_image_supremum_equiv, limit_sub_seq, sub_seq_eq_add_neg
from real.abs_conv import absolutely_converges, abs_fn, sub_seq, neg_seq, add_seq
from real.real_seq import converges, converges_to, limit

/// The Cauchy coefficient equals the diagonal function.
theorem cauchy_coefficient_eq_diagonal(a: Nat -> Real, b: Nat -> Real, n: Nat, k: Nat) {
    cauchy_coefficient(a, b, n)(k) = diagonal(prod_fn(a, b), n, k)
}

/// The Cauchy product equals a diagonal sum.
/// This shows that cauchy_product(a, b, n) = sum_{i=0}^n a(i) * b(n - i).
theorem cauchy_product_eq_diagonal(a: Nat -> Real, b: Nat -> Real, n: Nat) {
    cauchy_product(a, b, n) = diagonal_sum(prod_fn(a, b), n)
} by {
    // Both are defined as sums over the same function
    cauchy_product(a, b, n) = sum(map(n.suc.range, cauchy_coefficient(a, b, n)))
    diagonal_sum(prod_fn(a, b), n) = partial(diagonal(prod_fn(a, b), n), n.suc)
    partial(diagonal(prod_fn(a, b), n), n.suc) = sum(map(n.suc.range, diagonal(prod_fn(a, b), n)))

    // Show the functions are equal
    forall(k: Nat) {
        cauchy_coefficient(a, b, n)(k) = diagonal(prod_fn(a, b), n, k)
    }
    cauchy_coefficient(a, b, n) = diagonal(prod_fn(a, b), n)
}

/// The Cauchy product as a triangular sum.
/// For a fixed K, this computes the sum of all products a(i) * b(j) where i + j <= K.
define cauchy_triangular(a: Nat -> Real, b: Nat -> Real, k: Nat) -> Real {
    triangular_sum(prod_fn(a, b), k.suc)
}

/// The Cauchy product partial sum equals the triangular sum.
/// This shows that sum_{i=0}^k cauchy_product(a, b, i) = sum_{m+n<=k} a(m) * b(n).
theorem cauchy_product_eq_triangular(a: Nat -> Real, b: Nat -> Real, k: Nat) {
    partial(cauchy_seq(a, b), k.suc) = cauchy_triangular(a, b, k)
} by {
    // Use the fact that triangular_sum extends by diagonal_sum
    // and cauchy_product equals diagonal_sum

    cauchy_triangular(a, b, k) = triangular_sum(prod_fn(a, b), k.suc)

    // Prove by induction on k
    define p(n: Nat) -> Bool {
        partial(cauchy_seq(a, b), n.suc) = triangular_sum(prod_fn(a, b), n.suc)
    }

    // Base case: n = 0
    cauchy_seq(a, b, Nat.0) = cauchy_product(a, b, Nat.0)
    partial(cauchy_seq(a, b), Nat.1) = Real.0 + cauchy_seq(a, b, Nat.0)
    partial(cauchy_seq(a, b), Nat.1) = cauchy_product(a, b, Nat.0)

    cauchy_product(a, b, Nat.0) = diagonal_sum(prod_fn(a, b), Nat.0)

    // triangular_sum(prod_fn(a, b), 1) = triangular_sum(..., 0) + diagonal_sum(..., 0)
    triangular_sum(prod_fn(a, b), Nat.1) = triangular_sum(prod_fn(a, b), Nat.0) + diagonal_sum(prod_fn(a, b), Nat.0)
    triangular_sum(prod_fn(a, b), Nat.0) = Real.0
    triangular_sum(prod_fn(a, b), Nat.1) = diagonal_sum(prod_fn(a, b), Nat.0)

    partial(cauchy_seq(a, b), Nat.1) = triangular_sum(prod_fn(a, b), Nat.1)
    p(Nat.0)

    // Inductive step
    forall(n: Nat) {
        if p(n) {
            // IH: partial(cauchy_seq(a, b), n.suc) = triangular_sum(prod_fn(a, b), n.suc)
            // Want: partial(cauchy_seq(a, b), n.suc.suc) = triangular_sum(prod_fn(a, b), n.suc.suc)

            partial(cauchy_seq(a, b), n.suc.suc) = partial(cauchy_seq(a, b), n.suc) + cauchy_seq(a, b, n.suc)
            cauchy_seq(a, b, n.suc) = cauchy_product(a, b, n.suc)

            cauchy_product(a, b, n.suc) = diagonal_sum(prod_fn(a, b), n.suc)

            partial(cauchy_seq(a, b), n.suc.suc) = partial(cauchy_seq(a, b), n.suc) + diagonal_sum(prod_fn(a, b), n.suc)
            partial(cauchy_seq(a, b), n.suc.suc) = triangular_sum(prod_fn(a, b), n.suc) + diagonal_sum(prod_fn(a, b), n.suc)

            triangular_sum(prod_fn(a, b), n.suc.suc) = triangular_sum(prod_fn(a, b), n.suc) + diagonal_sum(prod_fn(a, b), n.suc)

            partial(cauchy_seq(a, b), n.suc.suc) = triangular_sum(prod_fn(a, b), n.suc.suc)
            p(n.suc)
        }
    }

    p(k)
    partial(cauchy_seq(a, b), k.suc) = triangular_sum(prod_fn(a, b), k.suc)
}

from real.triangular_sum import triangular_sum_converges_to_double_sum, abs_conv_pos_part_converges, abs_conv_neg_part_converges
from real.double_limit import doubly_increasing, double_image_is_supremum, double_image_is_supremum_from_parts, double_is_upper_bound, double_is_upper_bound_from_forall, doubly_increasing_bounded_converges, double_image, double_converges, double_converges_to
from real.real_series import increasing_convergent_bounded_by_limit, is_increasing, is_upper_bound, nonneg_seq, nonneg_imp_partial_increasing, partial_nonneg, tail, tail_imp_converges_to, add_seq_converges, neg_seq_converges
from real.double_sum import rectangular_sum_doubly_increasing
from real.real_seq import converges_to_imp_converges, converges_imp_converges_to, ub_imp_limit_lte, converges_to_unique, eventual_eq, eventual_ub, eq_converges, eq_imp_limit
from real.limits import converges_compose_suc

/// The absolute value of the product function equals the product of absolute values.
theorem abs_prod_fn(a: Nat -> Real, b: Nat -> Real, m: Nat, n: Nat) {
    abs_fn_2(prod_fn(a, b), m, n) = abs_fn(a, m) * abs_fn(b, n)
} by {
    a(m) * b(n) = prod_fn(a, b, m, n)
    a(m).abs * b(n).abs = (a(m) * b(n)).abs
    prod_fn(a, b, m, n).abs = abs_fn_2(prod_fn(a, b), m, n)
    a(m).abs = abs_fn(a, m)
    b(n).abs = abs_fn(b, n)
}

/// Rectangular sum of absolute product equals product of absolute partial sums.
theorem rectangular_abs_product(a: Nat -> Real, b: Nat -> Real, m: Nat, n: Nat) {
    rectangular_sum(abs_fn_2(prod_fn(a, b)), m, n) = partial(abs_fn(a), m) * partial(abs_fn(b), n)
} by {
    // abs_fn_2(prod_fn(a, b)) = prod_fn(abs_fn(a), abs_fn(b))
    forall(i: Nat, j: Nat) {
        abs_fn_2(prod_fn(a, b), i, j) = abs_fn(a, i) * abs_fn(b, j)
        prod_fn(abs_fn(a), abs_fn(b), i, j) = abs_fn(a, i) * abs_fn(b, j)
        abs_fn_2(prod_fn(a, b), i, j) = prod_fn(abs_fn(a), abs_fn(b), i, j)
    }
    abs_fn_2(prod_fn(a, b)) = prod_fn(abs_fn(a), abs_fn(b))

    rectangular_sum(abs_fn_2(prod_fn(a, b)), m, n) = rectangular_sum(prod_fn(abs_fn(a), abs_fn(b)), m, n)
    rectangular_sum(prod_fn(abs_fn(a), abs_fn(b)), m, n) = product_sum(abs_fn(a), abs_fn(b), m, n)
    product_sum(abs_fn(a), abs_fn(b), m, n) = partial(abs_fn(a), m) * partial(abs_fn(b), n)
}

from real.double_sum import square_sum, square_sum_converges_to_double_sum
from real.prod_seq import prod_seq, limit_prod_seq

/// The square sum of absolute product equals the product of partial sums.
theorem square_sum_abs_product(a: Nat -> Real, b: Nat -> Real, n: Nat) {
    square_sum(abs_fn_2(prod_fn(a, b)), n) = partial(abs_fn(a), n) * partial(abs_fn(b), n)
} by {
    square_sum(abs_fn_2(prod_fn(a, b)), n) = rectangular_sum(abs_fn_2(prod_fn(a, b)), n, n)
    rectangular_sum(abs_fn_2(prod_fn(a, b)), n, n) = partial(abs_fn(a), n) * partial(abs_fn(b), n)
}

/// When m <= n, rectangular sum is bounded by square sum of n.
theorem rectangular_bounded_by_square_case1(f: (Nat, Nat) -> Real, m: Nat, n: Nat) {
    nonneg_fn_2(f) and m <= n
    implies
    rectangular_sum(f, m, n) <= square_sum(f, n)
} by {
    if nonneg_fn_2(f) and m <= n {
        // Prove by induction on the distance n - m
        define p(k: Nat) -> Bool {
            forall(m0: Nat, n0: Nat) {
                nonneg_fn_2(f) and m0 + k = n0
                implies
                rectangular_sum(f, m0, n0) <= square_sum(f, n0)
            }
        }

        // Base case: k = 0, so m0 = n0
        forall(m0: Nat, n0: Nat) {
            if nonneg_fn_2(f) and m0 + Nat.0 = n0 {
                m0 = n0
                rectangular_sum(f, m0, n0) = rectangular_sum(f, n0, n0)
                rectangular_sum(f, n0, n0) = square_sum(f, n0)
                rectangular_sum(f, m0, n0) <= square_sum(f, n0)
            }
        }
        forall(m0: Nat, n0: Nat) {
            nonneg_fn_2(f) and m0 + Nat.0 = n0 implies rectangular_sum(f, m0, n0) <= square_sum(f, n0)
        }
        exists(k0: Nat, k1: Nat) {
            nonneg_fn_2(f) and k0 + Nat.0 = k1 and not rectangular_sum(f, k0, k1) <= square_sum(f, k1)
        } or p(Nat.0)
        p(Nat.0)

        // Inductive step
        forall(k: Nat) {
            if p(k) {
                forall(m0: Nat, n0: Nat) {
                    if nonneg_fn_2(f) and m0 + k.suc = n0 {
                        m0 + k < n0
                        m0.suc + k = n0

                        rectangular_sum(f, m0, n0) <= rectangular_sum(f, m0.suc, n0)

                        p(k)
                        nonneg_fn_2(f) and m0.suc + k = n0
                        rectangular_sum(f, m0.suc, n0) <= square_sum(f, n0)

                        rectangular_sum(f, m0, n0) <= square_sum(f, n0)
                    }
                }
                forall(m0: Nat, n0: Nat) {
                    nonneg_fn_2(f) and m0 + k.suc = n0 implies rectangular_sum(f, m0, n0) <= square_sum(f, n0)
                }
                exists(k0: Nat, k1: Nat) {
                    nonneg_fn_2(f) and k0 + k.suc = k1 and not rectangular_sum(f, k0, k1) <= square_sum(f, k1)
                } or p(k.suc)
                p(k.suc)
            }
        }

        // Apply the induction result
        m + (n - m) = n
        p(n - m)
        nonneg_fn_2(f) and m + (n - m) = n
        rectangular_sum(f, m, n) <= square_sum(f, n)
    }
}

/// When n <= m, rectangular sum is bounded by square sum of m.
theorem rectangular_bounded_by_square_case2(f: (Nat, Nat) -> Real, m: Nat, n: Nat) {
    nonneg_fn_2(f) and n <= m
    implies
    rectangular_sum(f, m, n) <= square_sum(f, m)
} by {
    if nonneg_fn_2(f) and n <= m {
        // Prove by induction on the distance m - n
        define q(k: Nat) -> Bool {
            forall(m0: Nat, n0: Nat) {
                nonneg_fn_2(f) and n0 + k = m0
                implies
                rectangular_sum(f, m0, n0) <= square_sum(f, m0)
            }
        }

        // Base case: k = 0, so n0 = m0
        forall(m0: Nat, n0: Nat) {
            if nonneg_fn_2(f) and n0 + Nat.0 = m0 {
                n0 = m0
                rectangular_sum(f, m0, n0) = rectangular_sum(f, m0, m0)
                rectangular_sum(f, m0, m0) = square_sum(f, m0)
                rectangular_sum(f, m0, n0) <= square_sum(f, m0)
            }
        }
        forall(m0: Nat, n0: Nat) {
            nonneg_fn_2(f) and n0 + Nat.0 = m0 implies rectangular_sum(f, m0, n0) <= square_sum(f, m0)
        }
        exists(k0: Nat, k1: Nat) {
            nonneg_fn_2(f) and k1 + Nat.0 = k0 and not rectangular_sum(f, k0, k1) <= square_sum(f, k0)
        } or q(Nat.0)
        q(Nat.0)

        // Inductive step
        forall(k: Nat) {
            if q(k) {
                forall(m0: Nat, n0: Nat) {
                    if nonneg_fn_2(f) and n0 + k.suc = m0 {
                        n0 + k < m0
                        n0.suc + k = m0

                        rectangular_sum(f, m0, n0) <= rectangular_sum(f, m0, n0.suc)

                        q(k)
                        nonneg_fn_2(f) and n0.suc + k = m0
                        rectangular_sum(f, m0, n0.suc) <= square_sum(f, m0)

                        rectangular_sum(f, m0, n0) <= square_sum(f, m0)
                    }
                }
                forall(m0: Nat, n0: Nat) {
                    nonneg_fn_2(f) and n0 + k.suc = m0 implies rectangular_sum(f, m0, n0) <= square_sum(f, m0)
                }
                exists(k0: Nat, k1: Nat) {
                    nonneg_fn_2(f) and k1 + k.suc = k0 and not rectangular_sum(f, k0, k1) <= square_sum(f, k0)
                } or q(k.suc)
                q(k.suc)
            }
        }

        // Apply the induction result
        n + (m - n) = m
        q(m - n)
        nonneg_fn_2(f) and n + (m - n) = m
        rectangular_sum(f, m, n) <= square_sum(f, m)
    }
}

/// If a and b converge absolutely, then their product converges as a double sum.
theorem product_abs_convergent(a: Nat -> Real, b: Nat -> Real) {
    absolutely_converges(a) and absolutely_converges(b)
    implies
    double_sum_converges(abs_fn_2(prod_fn(a, b)))
} by {
    if absolutely_converges(a) and absolutely_converges(b) {
        // First, show that abs_fn_2(prod_fn(a, b)) is nonnegative
        forall(m: Nat, n: Nat) {
            abs_fn_2(prod_fn(a, b), m, n) = abs_fn(a, m) * abs_fn(b, n)
            abs_fn(a, m) >= Real.0
            abs_fn(b, n) >= Real.0
            abs_fn_2(prod_fn(a, b), m, n) >= Real.0
        }
        nonneg_fn_2(abs_fn_2(prod_fn(a, b)))

        // Show that abs_fn_2(prod_fn(a, b)) is doubly increasing
        doubly_increasing(rectangular_sum(abs_fn_2(prod_fn(a, b))))

        // Show that the square_sum sequence converges
        absolutely_converges(a)
        converges(partial(abs_fn(a)))
        absolutely_converges(b)
        converges(partial(abs_fn(b)))

        // The square_sum equals the product of partial sums
        forall(n: Nat) {
            square_sum(abs_fn_2(prod_fn(a, b)), n) = partial(abs_fn(a), n) * partial(abs_fn(b), n)
            prod_seq(partial(abs_fn(a)), partial(abs_fn(b)), n) = partial(abs_fn(a), n) * partial(abs_fn(b), n)
            square_sum(abs_fn_2(prod_fn(a, b)), n) = prod_seq(partial(abs_fn(a)), partial(abs_fn(b)), n)
        }
        square_sum(abs_fn_2(prod_fn(a, b))) = prod_seq(partial(abs_fn(a)), partial(abs_fn(b)))

        // By limit_prod_seq, the product of convergent sequences converges
        converges(prod_seq(partial(abs_fn(a)), partial(abs_fn(b))))
        converges(square_sum(abs_fn_2(prod_fn(a, b))))

        // Get the limit value
        let sq_limit = limit(square_sum(abs_fn_2(prod_fn(a, b))))
        converges_to(square_sum(abs_fn_2(prod_fn(a, b))), sq_limit)

        // Show that square_sum is increasing
        forall(n: Nat) {
            rectangular_sum(abs_fn_2(prod_fn(a, b)), n, n) <= rectangular_sum(abs_fn_2(prod_fn(a, b)), n.suc, n)
            rectangular_sum(abs_fn_2(prod_fn(a, b)), n.suc, n) <= rectangular_sum(abs_fn_2(prod_fn(a, b)), n.suc, n.suc)

            square_sum(abs_fn_2(prod_fn(a, b)), n) = rectangular_sum(abs_fn_2(prod_fn(a, b)), n, n)
            square_sum(abs_fn_2(prod_fn(a, b)), n.suc) = rectangular_sum(abs_fn_2(prod_fn(a, b)), n.suc, n.suc)
            square_sum(abs_fn_2(prod_fn(a, b)), n) <= square_sum(abs_fn_2(prod_fn(a, b)), n.suc)
        }
        is_increasing(square_sum(abs_fn_2(prod_fn(a, b))))

        // Establish that sq_limit is the supremum of rectangular sums
        // First: sq_limit is an upper bound
        forall(m: Nat, n: Nat) {
            // Either m <= n or n <= m
            m <= n or n <= m

            if m <= n {
                rectangular_sum(abs_fn_2(prod_fn(a, b)), m, n) <= square_sum(abs_fn_2(prod_fn(a, b)), n)

                square_sum(abs_fn_2(prod_fn(a, b)), n) <= sq_limit
                rectangular_sum(abs_fn_2(prod_fn(a, b)), m, n) <= sq_limit
            }

            if n <= m {
                rectangular_sum(abs_fn_2(prod_fn(a, b)), m, n) <= square_sum(abs_fn_2(prod_fn(a, b)), m)

                square_sum(abs_fn_2(prod_fn(a, b)), m) <= sq_limit
                rectangular_sum(abs_fn_2(prod_fn(a, b)), m, n) <= sq_limit
            }

            rectangular_sum(abs_fn_2(prod_fn(a, b)), m, n) <= sq_limit
        }
        double_is_upper_bound(rectangular_sum(abs_fn_2(prod_fn(a, b))), sq_limit)

        // Second: any value less than sq_limit is not an upper bound
        forall(bound: Real) {
            if bound < sq_limit {
                // We prove "not double_is_upper_bound" by contradiction
                // Assume double_is_upper_bound holds
                if double_is_upper_bound(rectangular_sum(abs_fn_2(prod_fn(a, b))), bound) {
                    // Then all square_sum values are bounded by bound
                    forall(k: Nat) {
                        rectangular_sum(abs_fn_2(prod_fn(a, b)), k, k) <= bound
                        square_sum(abs_fn_2(prod_fn(a, b)), k) = rectangular_sum(abs_fn_2(prod_fn(a, b)), k, k)
                        square_sum(abs_fn_2(prod_fn(a, b)), k) <= bound
                    }
                    is_upper_bound(square_sum(abs_fn_2(prod_fn(a, b))), bound)

                    // Since square_sum is increasing and bounded by bound, we have sq_limit <= bound
                    converges(square_sum(abs_fn_2(prod_fn(a, b))))
                    forall(i: Nat) {
                        if Nat.0 <= i {
                            square_sum(abs_fn_2(prod_fn(a, b)), i) <= bound
                        }
                    }
                    exists(n0: Nat) {
                        forall(i: Nat) {
                            n0 <= i implies square_sum(abs_fn_2(prod_fn(a, b)), i) <= bound
                        }
                    }
                    eventual_ub(square_sum(abs_fn_2(prod_fn(a, b))), bound)
                    sq_limit <= bound

                    // This contradicts bound < sq_limit
                    sq_limit <= bound and bound < sq_limit
                }

                // Therefore the assumption was false
                not double_is_upper_bound(rectangular_sum(abs_fn_2(prod_fn(a, b))), bound)
            }
        }

        // Therefore sq_limit is the supremum
        double_image_is_supremum(rectangular_sum(abs_fn_2(prod_fn(a, b))), sq_limit)

        // Apply the doubly_increasing_bounded_converges theorem
        double_converges_to(rectangular_sum(abs_fn_2(prod_fn(a, b))), sq_limit)
        double_converges(rectangular_sum(abs_fn_2(prod_fn(a, b))))

        double_sum_converges(abs_fn_2(prod_fn(a, b)))
    }
}

from real.triangular_sum import pos_part_2, neg_part_2
from real.double_sum import double_sum_pos_part_converges_of_supremum, double_sum_neg_part_converges_of_supremum, rectangular_sum_pos_part_bounded_by_abs_supremum, rectangular_sum_neg_part_bounded_by_abs_supremum
from real.supremum import is_nonempty, is_set_upper_bound, has_upper_bound, is_set_supremum, completeness

/// Rectangular sum of product function equals product of partial sums.
theorem rectangular_product(a: Nat -> Real, b: Nat -> Real, m: Nat, n: Nat) {
    rectangular_sum(prod_fn(a, b), m, n) = partial(a, m) * partial(b, n)
} by {
    rectangular_sum(prod_fn(a, b), m, n) = product_sum(a, b, m, n)
    product_sum(a, b, m, n) = partial(a, m) * partial(b, n)
}

/// Square sum of product function equals product of partial sums.
theorem square_sum_product(a: Nat -> Real, b: Nat -> Real, n: Nat) {
    square_sum(prod_fn(a, b), n) = partial(a, n) * partial(b, n)
} by {
    square_sum(prod_fn(a, b), n) = rectangular_sum(prod_fn(a, b), n, n)
    rectangular_sum(prod_fn(a, b), n, n) = partial(a, n) * partial(b, n)
}

from real.double_limit import double_limit_sub, double_limit

/// If the absolute value function converges, then the function itself converges.
theorem double_sum_converges_from_abs(f: (Nat, Nat) -> Real, s: Real) {
    double_image_is_supremum(rectangular_sum(abs_fn_2(f)), s)
    implies
    double_sum_converges(f)
} by {
    if double_image_is_supremum(rectangular_sum(abs_fn_2(f)), s) {
        // Both positive and negative parts have convergent double sums
        double_sum_converges(pos_part_2(f))

        double_sum_converges(neg_part_2(f))

        // This means rectangular_sum converges for both parts
        double_converges(rectangular_sum(pos_part_2(f)))
        double_converges(rectangular_sum(neg_part_2(f)))

        // Use decomposition: rectangular_sum(f) = rectangular_sum(pos_part_2(f)) - rectangular_sum(neg_part_2(f))
        forall(m: Nat, n: Nat) {
            rectangular_sum(f, m, n) = rectangular_sum(pos_part_2(f), m, n) - rectangular_sum(neg_part_2(f), m, n)
            sub_fn_2(rectangular_sum(pos_part_2(f)), rectangular_sum(neg_part_2(f)), m, n) =
                rectangular_sum(pos_part_2(f), m, n) - rectangular_sum(neg_part_2(f), m, n)
            rectangular_sum(f, m, n) = sub_fn_2(rectangular_sum(pos_part_2(f)), rectangular_sum(neg_part_2(f)), m, n)
        }
        rectangular_sum(f) = sub_fn_2(rectangular_sum(pos_part_2(f)), rectangular_sum(neg_part_2(f)))

        // Apply double_limit_sub to get convergence of the difference
        double_converges_to(sub_fn_2(rectangular_sum(pos_part_2(f)), rectangular_sum(neg_part_2(f))),
            double_limit(rectangular_sum(pos_part_2(f))) - double_limit(rectangular_sum(neg_part_2(f))))

        double_converges_to(rectangular_sum(f),
            double_limit(rectangular_sum(pos_part_2(f))) - double_limit(rectangular_sum(neg_part_2(f))))
        double_converges(rectangular_sum(f))
        double_sum_converges(f)
    }
}

/// If the double sum converges (possibly from absolute convergence), then the triangular sum converges to the double sum.
theorem triangular_sum_converges_from_double_sum(f: (Nat, Nat) -> Real, s: Real) {
    double_image_is_supremum(rectangular_sum(abs_fn_2(f)), s)
    implies
    converges_to(triangular_sum(f), double_sum(f))
} by {
    if double_image_is_supremum(rectangular_sum(abs_fn_2(f)), s) {
        // Both positive and negative parts have convergent double sums
        double_sum_converges(pos_part_2(f))

        double_sum_converges(neg_part_2(f))

        // Both parts are nonnegative
        forall(i: Nat, j: Nat) {
            pos_part_2(f, i, j) >= Real.0
        }
        nonneg_fn_2(pos_part_2(f))

        forall(i: Nat, j: Nat) {
            neg_part_2(f, i, j) >= Real.0
        }
        nonneg_fn_2(neg_part_2(f))

        // Apply the nonnegative theorem to each part
        converges_to(triangular_sum(pos_part_2(f)), double_sum(pos_part_2(f)))

        converges_to(triangular_sum(neg_part_2(f)), double_sum(neg_part_2(f)))

        // Use decomposition
        forall(n: Nat) {
            triangular_sum(f, n) = triangular_sum(pos_part_2(f), n) - triangular_sum(neg_part_2(f), n)
            sub_seq(triangular_sum(pos_part_2(f)), triangular_sum(neg_part_2(f)), n) =
                triangular_sum(pos_part_2(f), n) - triangular_sum(neg_part_2(f), n)
            triangular_sum(f, n) = sub_seq(triangular_sum(pos_part_2(f)), triangular_sum(neg_part_2(f)), n)
        }
        triangular_sum(f) = sub_seq(triangular_sum(pos_part_2(f)), triangular_sum(neg_part_2(f)))

        // Show sub_seq converges
        converges(triangular_sum(pos_part_2(f)))
        converges(triangular_sum(neg_part_2(f)))

        limit(triangular_sum(pos_part_2(f))) = double_sum(pos_part_2(f))
        limit(triangular_sum(neg_part_2(f))) = double_sum(neg_part_2(f))

        // Convert sub_seq to add_seq with negation
        sub_seq(triangular_sum(pos_part_2(f)), triangular_sum(neg_part_2(f))) =
            add_seq(triangular_sum(pos_part_2(f)), neg_seq(triangular_sum(neg_part_2(f))))

        converges(neg_seq(triangular_sum(neg_part_2(f))))

        converges(add_seq(triangular_sum(pos_part_2(f)), neg_seq(triangular_sum(neg_part_2(f)))))
        converges(sub_seq(triangular_sum(pos_part_2(f)), triangular_sum(neg_part_2(f))))
        converges(triangular_sum(f))

        // Apply limit_sub_seq to get the limit
        limit(sub_seq(triangular_sum(pos_part_2(f)), triangular_sum(neg_part_2(f)))) =
            limit(triangular_sum(pos_part_2(f))) - limit(triangular_sum(neg_part_2(f)))
        limit(triangular_sum(f)) = limit(triangular_sum(pos_part_2(f))) - limit(triangular_sum(neg_part_2(f)))
        limit(triangular_sum(f)) = double_sum(pos_part_2(f)) - double_sum(neg_part_2(f))

        // Show double_sum(f) equals the difference of double sums
        double_sum_converges(f)

        // Show that double_sum(f) = double_sum(pos_part_2(f)) - double_sum(neg_part_2(f))
        // First show rectangular_sum(f) = sub_fn_2(rectangular_sum(pos_part_2(f)), rectangular_sum(neg_part_2(f)))
        forall(m: Nat, n: Nat) {
            rectangular_sum(f, m, n) = rectangular_sum(pos_part_2(f), m, n) - rectangular_sum(neg_part_2(f), m, n)
            sub_fn_2(rectangular_sum(pos_part_2(f)), rectangular_sum(neg_part_2(f)), m, n) =
                rectangular_sum(pos_part_2(f), m, n) - rectangular_sum(neg_part_2(f), m, n)
            rectangular_sum(f, m, n) = sub_fn_2(rectangular_sum(pos_part_2(f)), rectangular_sum(neg_part_2(f)), m, n)
        }
        rectangular_sum(f) = sub_fn_2(rectangular_sum(pos_part_2(f)), rectangular_sum(neg_part_2(f)))

        // Both pos and neg parts have convergent double sums (already shown)
        double_converges(rectangular_sum(pos_part_2(f)))
        double_converges(rectangular_sum(neg_part_2(f)))

        // Apply double_limit_sub to get double_converges_to for the difference
        double_converges_to(sub_fn_2(rectangular_sum(pos_part_2(f)), rectangular_sum(neg_part_2(f))),
            double_limit(rectangular_sum(pos_part_2(f))) - double_limit(rectangular_sum(neg_part_2(f))))

        // Since rectangular_sum(f) = sub_fn_2(...), they have the same limit
        double_converges_to(rectangular_sum(f),
            double_limit(rectangular_sum(pos_part_2(f))) - double_limit(rectangular_sum(neg_part_2(f))))

        double_limit(rectangular_sum(f)) = double_limit(rectangular_sum(pos_part_2(f))) - double_limit(rectangular_sum(neg_part_2(f)))
        double_sum(f) = double_sum(pos_part_2(f)) - double_sum(neg_part_2(f))

        limit(triangular_sum(f)) = double_sum(f)
        converges_to(triangular_sum(f), double_sum(f))
    }
}

/// If a and b converge absolutely, then the Cauchy product converges to the product of the sums.
/// This is the main theorem for the Cauchy product of absolutely convergent series.
theorem cauchy_product_converges(a: Nat -> Real, b: Nat -> Real) {
    absolutely_converges(a) and absolutely_converges(b)
    implies
    converges_to(partial(cauchy_seq(a, b)), limit(partial(a)) * limit(partial(b)))
} by {
    if absolutely_converges(a) and absolutely_converges(b) {
        // First, establish that the product has absolutely convergent double sum
        double_sum_converges(abs_fn_2(prod_fn(a, b)))

        // Extract the supremum by computing it from square_sum convergence
        // From the proof of product_abs_convergent, we know that square_sum converges
        forall(n: Nat) {
            square_sum(abs_fn_2(prod_fn(a, b)), n) = partial(abs_fn(a), n) * partial(abs_fn(b), n)
            prod_seq(partial(abs_fn(a)), partial(abs_fn(b)), n) = partial(abs_fn(a), n) * partial(abs_fn(b), n)
            square_sum(abs_fn_2(prod_fn(a, b)), n) = prod_seq(partial(abs_fn(a)), partial(abs_fn(b)), n)
        }
        square_sum(abs_fn_2(prod_fn(a, b))) = prod_seq(partial(abs_fn(a)), partial(abs_fn(b)))

        absolutely_converges(a)
        converges(partial(abs_fn(a)))
        absolutely_converges(b)
        converges(partial(abs_fn(b)))

        converges_to(prod_seq(partial(abs_fn(a)), partial(abs_fn(b))), limit(partial(abs_fn(a))) * limit(partial(abs_fn(b))))
        converges_to(square_sum(abs_fn_2(prod_fn(a, b))), limit(partial(abs_fn(a))) * limit(partial(abs_fn(b))))

        // The square sum converges to the double sum
        converges_to(square_sum(abs_fn_2(prod_fn(a, b))), double_sum(abs_fn_2(prod_fn(a, b))))

        // By uniqueness of limits
        limit(square_sum(abs_fn_2(prod_fn(a, b)))) = double_sum(abs_fn_2(prod_fn(a, b)))
        limit(square_sum(abs_fn_2(prod_fn(a, b)))) = limit(partial(abs_fn(a))) * limit(partial(abs_fn(b)))
        double_sum(abs_fn_2(prod_fn(a, b))) = limit(partial(abs_fn(a))) * limit(partial(abs_fn(b)))

        // Now we have the supremum value, and we can use double_sum_converges_from_abs
        // But we need to show that this value is indeed the supremum
        // This was proven in product_abs_convergent, so let's use that
        double_converges(rectangular_sum(abs_fn_2(prod_fn(a, b))))
        let abs_sup = double_limit(rectangular_sum(abs_fn_2(prod_fn(a, b))))
        double_converges_to(rectangular_sum(abs_fn_2(prod_fn(a, b))), abs_sup)

        // Show that abs_sup is the supremum
        forall(n: Nat) {
            rectangular_sum(abs_fn_2(prod_fn(a, b)), n, n) <= rectangular_sum(abs_fn_2(prod_fn(a, b)), n.suc, n)
            rectangular_sum(abs_fn_2(prod_fn(a, b)), n.suc, n) <= rectangular_sum(abs_fn_2(prod_fn(a, b)), n.suc, n.suc)

            square_sum(abs_fn_2(prod_fn(a, b)), n) = rectangular_sum(abs_fn_2(prod_fn(a, b)), n, n)
            square_sum(abs_fn_2(prod_fn(a, b)), n.suc) = rectangular_sum(abs_fn_2(prod_fn(a, b)), n.suc, n.suc)
            square_sum(abs_fn_2(prod_fn(a, b)), n) <= square_sum(abs_fn_2(prod_fn(a, b)), n.suc)
        }
        is_increasing(square_sum(abs_fn_2(prod_fn(a, b))))

        is_upper_bound(square_sum(abs_fn_2(prod_fn(a, b))), limit(square_sum(abs_fn_2(prod_fn(a, b)))))

        // Show abs_sup is the supremum
        forall(m: Nat, n: Nat) {
            m <= n or n <= m

            if m <= n {
                rectangular_sum(abs_fn_2(prod_fn(a, b)), m, n) <= square_sum(abs_fn_2(prod_fn(a, b)), n)
                square_sum(abs_fn_2(prod_fn(a, b)), n) <= limit(square_sum(abs_fn_2(prod_fn(a, b))))
                rectangular_sum(abs_fn_2(prod_fn(a, b)), m, n) <= limit(square_sum(abs_fn_2(prod_fn(a, b))))
            }

            if n <= m {
                rectangular_sum(abs_fn_2(prod_fn(a, b)), m, n) <= square_sum(abs_fn_2(prod_fn(a, b)), m)
                square_sum(abs_fn_2(prod_fn(a, b)), m) <= limit(square_sum(abs_fn_2(prod_fn(a, b))))
                rectangular_sum(abs_fn_2(prod_fn(a, b)), m, n) <= limit(square_sum(abs_fn_2(prod_fn(a, b))))
            }

            rectangular_sum(abs_fn_2(prod_fn(a, b)), m, n) <= limit(square_sum(abs_fn_2(prod_fn(a, b))))
        }
        forall(bound: Real) {
            if bound < limit(square_sum(abs_fn_2(prod_fn(a, b)))) {
                if double_is_upper_bound(rectangular_sum(abs_fn_2(prod_fn(a, b))), bound) {
                    forall(k: Nat) {
                        rectangular_sum(abs_fn_2(prod_fn(a, b)), k, k) <= bound
                        square_sum(abs_fn_2(prod_fn(a, b)), k) = rectangular_sum(abs_fn_2(prod_fn(a, b)), k, k)
                        square_sum(abs_fn_2(prod_fn(a, b)), k) <= bound
                    }
                    is_upper_bound(square_sum(abs_fn_2(prod_fn(a, b))), bound)

                    converges(square_sum(abs_fn_2(prod_fn(a, b))))
                    forall(i: Nat) {
                        if Nat.0 <= i {
                            square_sum(abs_fn_2(prod_fn(a, b)), i) <= bound
                        }
                    }
                    exists(n0: Nat) {
                        forall(i: Nat) {
                            n0 <= i implies square_sum(abs_fn_2(prod_fn(a, b)), i) <= bound
                        }
                    }
                    eventual_ub(square_sum(abs_fn_2(prod_fn(a, b))), bound)
                    limit(square_sum(abs_fn_2(prod_fn(a, b)))) <= bound

                    limit(square_sum(abs_fn_2(prod_fn(a, b)))) <= bound and bound < limit(square_sum(abs_fn_2(prod_fn(a, b))))
                }

                not double_is_upper_bound(rectangular_sum(abs_fn_2(prod_fn(a, b))), bound)
            }
        }

        double_is_upper_bound(rectangular_sum(abs_fn_2(prod_fn(a, b))), limit(square_sum(abs_fn_2(prod_fn(a, b)))))
        forall(bound: Real) {
            bound < limit(square_sum(abs_fn_2(prod_fn(a, b)))) implies not double_is_upper_bound(rectangular_sum(abs_fn_2(prod_fn(a, b))), bound)
        }
        double_is_upper_bound(rectangular_sum(abs_fn_2(prod_fn(a, b))), limit(square_sum(abs_fn_2(prod_fn(a, b))))) and forall(bound: Real) {
            bound < limit(square_sum(abs_fn_2(prod_fn(a, b)))) implies not double_is_upper_bound(rectangular_sum(abs_fn_2(prod_fn(a, b))), bound)
        }
        double_image_is_supremum(rectangular_sum(abs_fn_2(prod_fn(a, b))), limit(square_sum(abs_fn_2(prod_fn(a, b)))))
        double_image_is_supremum(rectangular_sum(abs_fn_2(prod_fn(a, b))), abs_sup)

        // Use our helper theorems to get convergence of the product itself and its triangular sum
        double_sum_converges(prod_fn(a, b))

        converges_to(triangular_sum(prod_fn(a, b)), double_sum(prod_fn(a, b)))

        // Show that the double sum equals the product of limits
        // The rectangular sums of the product converge to the product of limits
        forall(n: Nat) {
            square_sum(prod_fn(a, b), n) = partial(a, n) * partial(b, n)
            prod_seq(partial(a), partial(b), n) = partial(a, n) * partial(b, n)
            square_sum(prod_fn(a, b), n) = prod_seq(partial(a), partial(b), n)
        }
        square_sum(prod_fn(a, b)) = prod_seq(partial(a), partial(b))

        absolutely_converges(a)
        converges(partial(a))
        absolutely_converges(b)
        converges(partial(b))

        converges_to(prod_seq(partial(a), partial(b)), limit(partial(a)) * limit(partial(b)))
        converges_to(square_sum(prod_fn(a, b)), limit(partial(a)) * limit(partial(b)))

        // The square_sum converges to the product of limits
        limit(square_sum(prod_fn(a, b))) = limit(partial(a)) * limit(partial(b))

        // The double sum equals the limit of square_sum
        converges_to(square_sum(prod_fn(a, b)), double_sum(prod_fn(a, b)))
        limit(square_sum(prod_fn(a, b))) = double_sum(prod_fn(a, b))

        double_sum(prod_fn(a, b)) = limit(partial(a)) * limit(partial(b))

        // Therefore triangular_sum converges to the product of limits
        converges_to(triangular_sum(prod_fn(a, b)), limit(partial(a)) * limit(partial(b)))

        // Now use cauchy_product_eq_triangular to relate to Cauchy partial sums
        forall(k: Nat) {
            partial(cauchy_seq(a, b), k.suc) = triangular_sum(prod_fn(a, b), k.suc)
            compose(partial(cauchy_seq(a, b)), Nat.suc, k) = triangular_sum(prod_fn(a, b), k.suc)
            compose(triangular_sum(prod_fn(a, b)), Nat.suc, k) = triangular_sum(prod_fn(a, b), k.suc)
            compose(partial(cauchy_seq(a, b)), Nat.suc, k) = compose(triangular_sum(prod_fn(a, b)), Nat.suc, k)
        }
        compose(partial(cauchy_seq(a, b)), Nat.suc) = compose(triangular_sum(prod_fn(a, b)), Nat.suc)

        // Since triangular_sum converges, so does its compose
        converges(triangular_sum(prod_fn(a, b)))
        limit(triangular_sum(prod_fn(a, b))) = limit(partial(a)) * limit(partial(b))

        converges_to(compose(triangular_sum(prod_fn(a, b)), Nat.suc), limit(triangular_sum(prod_fn(a, b))))
        converges_to(compose(triangular_sum(prod_fn(a, b)), Nat.suc), limit(partial(a)) * limit(partial(b)))
        converges_to(compose(partial(cauchy_seq(a, b)), Nat.suc), limit(partial(a)) * limit(partial(b)))

        // compose(a, Nat.suc) = compose(a, Nat.1.add) = tail(a, 1)
        compose(partial(cauchy_seq(a, b)), Nat.suc) = compose(partial(cauchy_seq(a, b)), Nat.1.add)
        forall(i: Nat) {
            compose(partial(cauchy_seq(a, b)), Nat.1.add, i) = partial(cauchy_seq(a, b), Nat.1 + i)
            tail(partial(cauchy_seq(a, b)), Nat.1, i) = partial(cauchy_seq(a, b), Nat.1 + i)
            compose(partial(cauchy_seq(a, b)), Nat.1.add, i) = tail(partial(cauchy_seq(a, b)), Nat.1, i)
        }
        compose(partial(cauchy_seq(a, b)), Nat.1.add) = tail(partial(cauchy_seq(a, b)), Nat.1)
        compose(partial(cauchy_seq(a, b)), Nat.suc) = tail(partial(cauchy_seq(a, b)), Nat.1)

        converges_to(tail(partial(cauchy_seq(a, b)), Nat.1), limit(partial(a)) * limit(partial(b)))
        converges(tail(partial(cauchy_seq(a, b)), Nat.1))
        limit(tail(partial(cauchy_seq(a, b)), Nat.1)) = limit(partial(a)) * limit(partial(b))

        // Use tail_imp_converges_to: if tail converges, then the original sequence converges to the same limit
        converges_to(partial(cauchy_seq(a, b)), limit(tail(partial(cauchy_seq(a, b)), Nat.1)))
        converges_to(partial(cauchy_seq(a, b)), limit(partial(a)) * limit(partial(b)))
    }
}
