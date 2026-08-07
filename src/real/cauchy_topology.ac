from nat import Nat
from data.basic.set import Set
from real.real_field import Real
from real.real_seq import converges, converges_to, limit
from real.bounded_seq import converges_imp_bounded_seq, converges_to_imp_bounded_seq,
    is_bounded_seq
from real.cauchy_criterion import cauchy_imp_converges, cauchy_imp_converges_to_limit,
    cauchy_imp_exists_limit, is_cauchy_seq
from real.sequence_cluster_sets import cluster_point_in_range_closure,
    cluster_set_subset_closed_set_of_eventually_in, cluster_set_subset_closed_set_of_seq_in,
    convergent_sequence_cluster_set_nonempty, convergent_sequence_limit_in_cluster_set,
    sequence_cluster_set
from real.sequence_set_membership import seq_eventually_in_real_set, seq_frequently_in_real_set,
    seq_in_real_set
from real.sequence_tail_sets import sequence_range_set
from real.topology import closure, is_closed_set
from real.topology_sequence_closure import closed_set_contains_eventual_seq_limit,
    closed_set_contains_frequent_seq_limit, closed_set_contains_seq_in_real_set_limit,
    seq_eventually_converges_imp_closure_contains,
    seq_frequently_converges_imp_closure_contains, seq_in_real_set_converges_imp_closure_contains

/// A Cauchy real sequence is bounded.
theorem cauchy_seq_is_bounded(a: Nat -> Real) {
    is_cauchy_seq(a) implies is_bounded_seq(a)
} by {
    if is_cauchy_seq(a) {
        cauchy_imp_converges(a)
        converges(a)
        converges_imp_bounded_seq(a)
        is_bounded_seq(a)
    }
}

/// A Cauchy real sequence is bounded through its canonical limit.
theorem cauchy_seq_bounded_by_limit_convergence(a: Nat -> Real) {
    is_cauchy_seq(a) implies is_bounded_seq(a)
} by {
    if is_cauchy_seq(a) {
        cauchy_imp_converges_to_limit(a)
        converges_to(a, limit(a))
        converges_to_imp_bounded_seq(a, limit(a))
        is_bounded_seq(a)
    }
}

/// A Cauchy real sequence has its canonical limit in its cluster set.
theorem cauchy_seq_limit_in_cluster_set(a: Nat -> Real) {
    is_cauchy_seq(a) implies sequence_cluster_set(a).contains(limit(a))
} by {
    if is_cauchy_seq(a) {
        cauchy_imp_converges_to_limit(a)
        converges_to(a, limit(a))
        convergent_sequence_limit_in_cluster_set(a, limit(a))
        sequence_cluster_set(a).contains(limit(a))
    }
}

/// The cluster set of a Cauchy real sequence is nonempty.
theorem cauchy_seq_cluster_set_nonempty(a: Nat -> Real) {
    is_cauchy_seq(a) implies exists(x: Real) { sequence_cluster_set(a).contains(x) }
} by {
    if is_cauchy_seq(a) {
        cauchy_imp_converges_to_limit(a)
        converges_to(a, limit(a))
        convergent_sequence_cluster_set_nonempty(a, limit(a))
        exists(x: Real) {
            sequence_cluster_set(a).contains(x)
        }
    }
}

/// The canonical limit of a Cauchy real sequence lies in the closure of its range.
theorem cauchy_seq_limit_in_range_closure(a: Nat -> Real) {
    is_cauchy_seq(a) implies closure(sequence_range_set(a)).contains(limit(a))
} by {
    if is_cauchy_seq(a) {
        cauchy_seq_limit_in_cluster_set(a)
        sequence_cluster_set(a).contains(limit(a))
        cluster_point_in_range_closure(a, limit(a))
        closure(sequence_range_set(a)).contains(limit(a))
    }
}

/// A Cauchy real sequence has some limit in its cluster set.
theorem cauchy_seq_has_cluster_limit(a: Nat -> Real) {
    is_cauchy_seq(a) implies exists(x: Real) {
        converges_to(a, x) and sequence_cluster_set(a).contains(x)
    }
} by {
    if is_cauchy_seq(a) {
        cauchy_imp_exists_limit(a)
        let x: Real satisfy {
            converges_to(a, x)
        }
        convergent_sequence_limit_in_cluster_set(a, x)
        sequence_cluster_set(a).contains(x)
        exists(y: Real) {
            converges_to(a, y) and sequence_cluster_set(a).contains(y)
        }
    }
}

/// Frequent membership along a Cauchy real sequence puts its canonical limit in the closure.
theorem cauchy_seq_frequently_in_imp_limit_in_closure(s: Set[Real], a: Nat -> Real) {
    seq_frequently_in_real_set(s, a) and is_cauchy_seq(a) implies closure(s).contains(limit(a))
} by {
    if seq_frequently_in_real_set(s, a) and is_cauchy_seq(a) {
        cauchy_imp_converges_to_limit(a)
        converges_to(a, limit(a))
        seq_frequently_converges_imp_closure_contains(s, a, limit(a))
        closure(s).contains(limit(a))
    }
}

/// Eventual membership along a Cauchy real sequence puts its canonical limit in the closure.
theorem cauchy_seq_eventually_in_imp_limit_in_closure(s: Set[Real], a: Nat -> Real) {
    seq_eventually_in_real_set(s, a) and is_cauchy_seq(a) implies closure(s).contains(limit(a))
} by {
    if seq_eventually_in_real_set(s, a) and is_cauchy_seq(a) {
        cauchy_imp_converges_to_limit(a)
        converges_to(a, limit(a))
        seq_eventually_converges_imp_closure_contains(s, a, limit(a))
        closure(s).contains(limit(a))
    }
}

/// Pointwise membership along a Cauchy real sequence puts its canonical limit in the closure.
theorem cauchy_seq_in_real_set_imp_limit_in_closure(s: Set[Real], a: Nat -> Real) {
    seq_in_real_set(s, a) and is_cauchy_seq(a) implies closure(s).contains(limit(a))
} by {
    if seq_in_real_set(s, a) and is_cauchy_seq(a) {
        cauchy_imp_converges_to_limit(a)
        converges_to(a, limit(a))
        seq_in_real_set_converges_imp_closure_contains(s, a, limit(a))
        closure(s).contains(limit(a))
    }
}

/// A closed set contains the canonical limit of every frequently visiting Cauchy sequence.
theorem closed_set_contains_frequent_cauchy_seq_limit(s: Set[Real], a: Nat -> Real) {
    is_closed_set(s) and seq_frequently_in_real_set(s, a) and is_cauchy_seq(a)
    implies s.contains(limit(a))
} by {
    if is_closed_set(s) and seq_frequently_in_real_set(s, a) and is_cauchy_seq(a) {
        cauchy_imp_converges_to_limit(a)
        converges_to(a, limit(a))
        closed_set_contains_frequent_seq_limit(s, a, limit(a))
        s.contains(limit(a))
    }
}

/// A closed set contains the canonical limit of every eventually contained Cauchy sequence.
theorem closed_set_contains_eventual_cauchy_seq_limit(s: Set[Real], a: Nat -> Real) {
    is_closed_set(s) and seq_eventually_in_real_set(s, a) and is_cauchy_seq(a)
    implies s.contains(limit(a))
} by {
    if is_closed_set(s) and seq_eventually_in_real_set(s, a) and is_cauchy_seq(a) {
        cauchy_imp_converges_to_limit(a)
        converges_to(a, limit(a))
        closed_set_contains_eventual_seq_limit(s, a, limit(a))
        s.contains(limit(a))
    }
}

/// A closed set contains the canonical limit of every contained Cauchy sequence.
theorem closed_set_contains_cauchy_seq_in_real_set_limit(s: Set[Real], a: Nat -> Real) {
    is_closed_set(s) and seq_in_real_set(s, a) and is_cauchy_seq(a) implies s.contains(limit(a))
} by {
    if is_closed_set(s) and seq_in_real_set(s, a) and is_cauchy_seq(a) {
        cauchy_imp_converges_to_limit(a)
        converges_to(a, limit(a))
        closed_set_contains_seq_in_real_set_limit(s, a, limit(a))
        s.contains(limit(a))
    }
}

/// The cluster set of an eventually contained Cauchy sequence is contained in a closed set.
theorem cauchy_cluster_set_subset_closed_set_of_eventually_in(a: Nat -> Real, s: Set[Real]) {
    is_cauchy_seq(a) and seq_eventually_in_real_set(s, a) and is_closed_set(s)
    implies sequence_cluster_set(a).subset(s)
} by {
    if is_cauchy_seq(a) and seq_eventually_in_real_set(s, a) and is_closed_set(s) {
        cluster_set_subset_closed_set_of_eventually_in(a, s)
        sequence_cluster_set(a).subset(s)
    }
}

/// The cluster set of a contained Cauchy sequence is contained in a closed set.
theorem cauchy_cluster_set_subset_closed_set_of_seq_in(a: Nat -> Real, s: Set[Real]) {
    is_cauchy_seq(a) and seq_in_real_set(s, a) and is_closed_set(s)
    implies sequence_cluster_set(a).subset(s)
} by {
    if is_cauchy_seq(a) and seq_in_real_set(s, a) and is_closed_set(s) {
        cluster_set_subset_closed_set_of_seq_in(a, s)
        sequence_cluster_set(a).subset(s)
    }
}
