/// Topological consumers of eventual sequence tail/subsequence transport.
///
/// This module intentionally does not introduce asymptotic notation or another
/// eventual predicate API.  It packages closure and closed-set consequences that
/// consume the accepted Nat-eventual, sequence-eventual, and reindexing transport
/// lemmas.

from data.basic.functions import compose
from nat import Nat
from data.basic.set import Set
from real.real_field import Real
from real.real_seq import converges_to
from real.topology import closure, is_closed_set
from real.limits import tends_to_infinity, is_subsequence_index, subsequence
from real.sequence_set_membership import seq_eventually_in_real_set
from real.sequence_eventual_bridge import seq_eventually_equal_real,
    seq_eventually_equal_real_preserves_eventual_membership
from real.sequence_eventual_tail_transport import seq_eventually_in_real_set_compose_tends_to_infinity,
    seq_eventually_in_real_set_shift_add,
    seq_eventually_in_real_set_subsequence,
    seq_eventually_equal_real_compose_tends_to_infinity,
    seq_eventually_equal_real_shift_add,
    seq_eventually_equal_real_subsequence
from real.topology_sequence_closure import seq_eventually_converges_imp_closure_contains,
    closed_set_contains_eventual_seq_limit

/// If a sequence is eventually in `s`, every convergent reindexing along a map
/// tending to infinity has its limit in `closure(s)`.
theorem eventual_compose_tends_to_infinity_limit_in_closure(
    s: Set[Real],
    a: Nat -> Real,
    f: Nat -> Nat,
    x: Real
) {
    seq_eventually_in_real_set(s, a) and tends_to_infinity(f) and converges_to(compose(a, f), x)
    implies closure(s).contains(x)
} by {
    if seq_eventually_in_real_set(s, a) and tends_to_infinity(f) and converges_to(compose(a, f), x) {
        seq_eventually_in_real_set_compose_tends_to_infinity(s, a, f)
        seq_eventually_in_real_set(s, compose(a, f))
        seq_eventually_converges_imp_closure_contains(s, compose(a, f), x)
        closure(s).contains(x)
    }
}

/// Closed-set variant of `eventual_compose_tends_to_infinity_limit_in_closure`.
theorem closed_set_contains_eventual_compose_tends_to_infinity_limit(
    s: Set[Real],
    a: Nat -> Real,
    f: Nat -> Nat,
    x: Real
) {
    is_closed_set(s) and seq_eventually_in_real_set(s, a) and tends_to_infinity(f) and
    converges_to(compose(a, f), x)
    implies s.contains(x)
} by {
    if is_closed_set(s) and seq_eventually_in_real_set(s, a) and tends_to_infinity(f) and
    converges_to(compose(a, f), x) {
        seq_eventually_in_real_set_compose_tends_to_infinity(s, a, f)
        seq_eventually_in_real_set(s, compose(a, f))
        closed_set_contains_eventual_seq_limit(s, compose(a, f), x)
        s.contains(x)
    }
}

/// If a sequence is eventually in `s`, every convergent finite-prefix shift has
/// its limit in `closure(s)`.
theorem eventual_shift_add_limit_in_closure(s: Set[Real], a: Nat -> Real, k: Nat, x: Real) {
    seq_eventually_in_real_set(s, a) and converges_to(compose(a, k.add), x)
    implies closure(s).contains(x)
} by {
    if seq_eventually_in_real_set(s, a) and converges_to(compose(a, k.add), x) {
        seq_eventually_in_real_set_shift_add(s, a, k)
        seq_eventually_in_real_set(s, compose(a, k.add))
        seq_eventually_converges_imp_closure_contains(s, compose(a, k.add), x)
        closure(s).contains(x)
    }
}

/// Closed-set variant for a convergent finite-prefix shift.
theorem closed_set_contains_eventual_shift_add_limit(
    s: Set[Real],
    a: Nat -> Real,
    k: Nat,
    x: Real
) {
    is_closed_set(s) and seq_eventually_in_real_set(s, a) and converges_to(compose(a, k.add), x)
    implies s.contains(x)
} by {
    if is_closed_set(s) and seq_eventually_in_real_set(s, a) and converges_to(compose(a, k.add), x) {
        seq_eventually_in_real_set_shift_add(s, a, k)
        seq_eventually_in_real_set(s, compose(a, k.add))
        closed_set_contains_eventual_seq_limit(s, compose(a, k.add), x)
        s.contains(x)
    }
}

/// If a sequence is eventually in `s`, every convergent selected subsequence has
/// its limit in `closure(s)`.
theorem eventual_subsequence_limit_in_closure(
    s: Set[Real],
    a: Nat -> Real,
    f: Nat -> Nat,
    x: Real
) {
    seq_eventually_in_real_set(s, a) and is_subsequence_index(f) and converges_to(subsequence(a, f), x)
    implies closure(s).contains(x)
} by {
    if seq_eventually_in_real_set(s, a) and is_subsequence_index(f) and converges_to(subsequence(a, f), x) {
        seq_eventually_in_real_set_subsequence(s, a, f)
        seq_eventually_in_real_set(s, subsequence(a, f))
        seq_eventually_converges_imp_closure_contains(s, subsequence(a, f), x)
        closure(s).contains(x)
    }
}

/// Closed-set variant for a convergent selected subsequence.
theorem closed_set_contains_eventual_subsequence_limit(
    s: Set[Real],
    a: Nat -> Real,
    f: Nat -> Nat,
    x: Real
) {
    is_closed_set(s) and seq_eventually_in_real_set(s, a) and is_subsequence_index(f) and
    converges_to(subsequence(a, f), x)
    implies s.contains(x)
} by {
    if is_closed_set(s) and seq_eventually_in_real_set(s, a) and is_subsequence_index(f) and
    converges_to(subsequence(a, f), x) {
        seq_eventually_in_real_set_subsequence(s, a, f)
        seq_eventually_in_real_set(s, subsequence(a, f))
        closed_set_contains_eventual_seq_limit(s, subsequence(a, f), x)
        s.contains(x)
    }
}

/// Eventual equality lets a convergent reindexing of `b` inherit closure
/// membership from eventual membership of `a` in `s`.
theorem eventually_equal_compose_tends_to_infinity_limit_in_closure(
    s: Set[Real],
    a: Nat -> Real,
    b: Nat -> Real,
    f: Nat -> Nat,
    x: Real
) {
    seq_eventually_in_real_set(s, a) and seq_eventually_equal_real(a, b) and tends_to_infinity(f) and
    converges_to(compose(b, f), x)
    implies closure(s).contains(x)
} by {
    if seq_eventually_in_real_set(s, a) and seq_eventually_equal_real(a, b) and tends_to_infinity(f) and
    converges_to(compose(b, f), x) {
        seq_eventually_in_real_set_compose_tends_to_infinity(s, a, f)
        seq_eventually_in_real_set(s, compose(a, f))
        seq_eventually_equal_real_compose_tends_to_infinity(a, b, f)
        seq_eventually_equal_real(compose(a, f), compose(b, f))
        seq_eventually_equal_real_preserves_eventual_membership(s, compose(a, f), compose(b, f))
        seq_eventually_in_real_set(s, compose(b, f))
        seq_eventually_converges_imp_closure_contains(s, compose(b, f), x)
        closure(s).contains(x)
    }
}

/// Closed-set variant of eventual equality plus reindexing along a map tending to infinity.
theorem closed_set_contains_eventually_equal_compose_tends_to_infinity_limit(
    s: Set[Real],
    a: Nat -> Real,
    b: Nat -> Real,
    f: Nat -> Nat,
    x: Real
) {
    is_closed_set(s) and seq_eventually_in_real_set(s, a) and seq_eventually_equal_real(a, b) and
    tends_to_infinity(f) and converges_to(compose(b, f), x)
    implies s.contains(x)
} by {
    if is_closed_set(s) and seq_eventually_in_real_set(s, a) and seq_eventually_equal_real(a, b) and
    tends_to_infinity(f) and converges_to(compose(b, f), x) {
        seq_eventually_in_real_set_compose_tends_to_infinity(s, a, f)
        seq_eventually_in_real_set(s, compose(a, f))
        seq_eventually_equal_real_compose_tends_to_infinity(a, b, f)
        seq_eventually_equal_real(compose(a, f), compose(b, f))
        seq_eventually_equal_real_preserves_eventual_membership(s, compose(a, f), compose(b, f))
        seq_eventually_in_real_set(s, compose(b, f))
        closed_set_contains_eventual_seq_limit(s, compose(b, f), x)
        s.contains(x)
    }
}

/// Eventual equality lets a convergent finite-prefix shift of `b` inherit closure
/// membership from eventual membership of `a` in `s`.
theorem eventually_equal_shift_add_limit_in_closure(
    s: Set[Real],
    a: Nat -> Real,
    b: Nat -> Real,
    k: Nat,
    x: Real
) {
    seq_eventually_in_real_set(s, a) and seq_eventually_equal_real(a, b) and
    converges_to(compose(b, k.add), x)
    implies closure(s).contains(x)
} by {
    if seq_eventually_in_real_set(s, a) and seq_eventually_equal_real(a, b) and
    converges_to(compose(b, k.add), x) {
        seq_eventually_in_real_set_shift_add(s, a, k)
        seq_eventually_in_real_set(s, compose(a, k.add))
        seq_eventually_equal_real_shift_add(a, b, k)
        seq_eventually_equal_real(compose(a, k.add), compose(b, k.add))
        seq_eventually_equal_real_preserves_eventual_membership(s, compose(a, k.add), compose(b, k.add))
        seq_eventually_in_real_set(s, compose(b, k.add))
        seq_eventually_converges_imp_closure_contains(s, compose(b, k.add), x)
        closure(s).contains(x)
    }
}

/// Closed-set variant for eventual equality and a convergent finite-prefix shift.
theorem closed_set_contains_eventually_equal_shift_add_limit(
    s: Set[Real],
    a: Nat -> Real,
    b: Nat -> Real,
    k: Nat,
    x: Real
) {
    is_closed_set(s) and seq_eventually_in_real_set(s, a) and seq_eventually_equal_real(a, b) and
    converges_to(compose(b, k.add), x)
    implies s.contains(x)
} by {
    if is_closed_set(s) and seq_eventually_in_real_set(s, a) and seq_eventually_equal_real(a, b) and
    converges_to(compose(b, k.add), x) {
        seq_eventually_in_real_set_shift_add(s, a, k)
        seq_eventually_in_real_set(s, compose(a, k.add))
        seq_eventually_equal_real_shift_add(a, b, k)
        seq_eventually_equal_real(compose(a, k.add), compose(b, k.add))
        seq_eventually_equal_real_preserves_eventual_membership(s, compose(a, k.add), compose(b, k.add))
        seq_eventually_in_real_set(s, compose(b, k.add))
        closed_set_contains_eventual_seq_limit(s, compose(b, k.add), x)
        s.contains(x)
    }
}

/// Eventual equality lets a convergent selected subsequence of `b` inherit
/// closure membership from eventual membership of `a` in `s`.
theorem eventually_equal_subsequence_limit_in_closure(
    s: Set[Real],
    a: Nat -> Real,
    b: Nat -> Real,
    f: Nat -> Nat,
    x: Real
) {
    seq_eventually_in_real_set(s, a) and seq_eventually_equal_real(a, b) and is_subsequence_index(f) and
    converges_to(subsequence(b, f), x)
    implies closure(s).contains(x)
} by {
    if seq_eventually_in_real_set(s, a) and seq_eventually_equal_real(a, b) and is_subsequence_index(f) and
    converges_to(subsequence(b, f), x) {
        seq_eventually_in_real_set_subsequence(s, a, f)
        seq_eventually_in_real_set(s, subsequence(a, f))
        seq_eventually_equal_real_subsequence(a, b, f)
        seq_eventually_equal_real(subsequence(a, f), subsequence(b, f))
        seq_eventually_equal_real_preserves_eventual_membership(s, subsequence(a, f), subsequence(b, f))
        seq_eventually_in_real_set(s, subsequence(b, f))
        seq_eventually_converges_imp_closure_contains(s, subsequence(b, f), x)
        closure(s).contains(x)
    }
}

/// Closed-set variant for eventual equality and a convergent selected subsequence.
theorem closed_set_contains_eventually_equal_subsequence_limit(
    s: Set[Real],
    a: Nat -> Real,
    b: Nat -> Real,
    f: Nat -> Nat,
    x: Real
) {
    is_closed_set(s) and seq_eventually_in_real_set(s, a) and seq_eventually_equal_real(a, b) and
    is_subsequence_index(f) and converges_to(subsequence(b, f), x)
    implies s.contains(x)
} by {
    if is_closed_set(s) and seq_eventually_in_real_set(s, a) and seq_eventually_equal_real(a, b) and
    is_subsequence_index(f) and converges_to(subsequence(b, f), x) {
        seq_eventually_in_real_set_subsequence(s, a, f)
        seq_eventually_in_real_set(s, subsequence(a, f))
        seq_eventually_equal_real_subsequence(a, b, f)
        seq_eventually_equal_real(subsequence(a, f), subsequence(b, f))
        seq_eventually_equal_real_preserves_eventual_membership(s, subsequence(a, f), subsequence(b, f))
        seq_eventually_in_real_set(s, subsequence(b, f))
        closed_set_contains_eventual_seq_limit(s, subsequence(b, f), x)
        s.contains(x)
    }
}
