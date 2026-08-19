/// Abel's theorem for real power series.
///
/// If the series Σ a(n) converges, then the power series Σ a(n) x^n has
/// limit Σ a(n) as x approaches 1 from below:
///
///     lim_{x -> 1^-} Σ a(n) x^n = Σ a(n).
///
/// The formal statement is sequential: for every sequence q with 0 <= q(n) < 1
/// and q(n) -> 1, the sums ps_sum(a, q(n)) converge to limit(partial(a))
/// (abel_theorem).
///
/// The proof follows the classical route:
///
///   (1) the finite Abel identity: with s_n = Σ_{k<n} a(k),
///       Σ_{k<n} a(k) x^k = (1 - x) Σ_{k<n} s_{k+1} x^k + s_n x^n,
///   (2) the infinite form: ps_sum(a, x) - s = (1 - x) · L where
///       L = lim Σ_{k} (s_{k+1} - s) x^k,
///   (3) the deviation tail |s_{k+1} - s| < eps from some N on gives
///       |ps_sum(a, x) - s| <= (1 - x) · Σ_{k<N} |s_{k+1} - s| + eps · x^N,
///       which tends to eps as x -> 1^- (abel_estimate),
///   (4) the sequential limit transfers this to q(n) -> 1.

from nat import Nat, from_nat, pow_zero, pow_add, one_pow, lte_add_left, lt_suc
from list import partial, partial_pointwise_eq
from data.basic.functions import compose
from data.basic.logic import eq_true_intro, eq_true_elim
from real.real_field import Real, mul_div_cancel, div_le_of_mul_le, zero_is_different_than_one, geom_series
from real.real_base import abs_gte_zero, lte_lt_trans, lte_abs, add_zero_right, lt_add_pos, gt_zero_imp_pos, pos_gt_zero, add_lte_add, neg_neg, neg_distrib, abs_neg, lt_add_right, lte_add_right, lte_self, add_assoc, add_comm, sub_cancels
from real.real_ring import converges, limit, converges_to, mul_abs, real_mul_comm, mul_nonneg, mul_sub_distrib_right, mul_distrib_left, mul_sub_distrib_left, mul_zero_right, mul_neg_left
from real.real_seq import converges_to_imp_converges, converges_imp_converges_to, converges_to_unique, ub_imp_limit_lte, eventual_ub, tail_bound, tail_bound_implies_is_close, limit_add_seq
from real.real_series import geom_converges, mul_seq_converges_to, add_seq_converges, partial_suc, partial_zero, partial_mul_seq_comm, partial_add_seq_comm, series_conv_imp_term_vanishes, partial_seq_lte, pos_geom_indirect_upper_bound, tail_converges_to, partial_tail_decomp, mul_seq, add_seq, pow_nonneg, seq_lte, tail_partial_converges, tail
from real.limits import limit_abs_seq, abs_seq
from real.limit_theorems import limit_of_product
from real.abs_conv import absolutely_converges, abs_fn, absolutely_converges_imp_converges
from real.series_deep import power_series_term, power_series_abs_conv_inside_radius, abs_pow_abs
from real.power_series_analysis import ps_sum, ps_term, converges_pointwise_eq, limit_pointwise_eq
from real.power_series_radius import ps_term_eq_power_series_term
from real.series_geometric_majorant import geometric_majorant, eventually_abs_le_geometric_absolutely_converges
from real.bounded_seq import is_bounded_seq, converges_to_imp_bounded_seq
from real.prod_seq import prod_seq
from real.derivative_exp_log import partial_abs_le_partial_abs_fn
from real.exp import two, two_positive, pow_suc
from real.series_root_test import pow_lte_base
from real.am_gm import div_pos_of_pos_pos
from real.holder_minkowski import abs_eq_self_of_nonneg
from real.integrability import lt_imp_pos_sub
from order import lt_iff_lte_and_ne, not_gt_imp_lte, lte_antisymm, lt_of_lt_of_lte, lt_imp_lte, lte_trans
from ordered_field import mul_le_mul_of_nonneg_right, mul_le_mul_of_nonneg_left, mul_lt_mul_of_pos_right

numerals Real
numerals Nat

/// The Abel sequence s_{n+1} x^n of the partial sums of a.
define abel_seq(a: Nat -> Real, x: Real, n: Nat) -> Real {
    partial(a, n.suc) * x.pow(n)
}

/// The centered Abel sequence (s_{n+1} - s) x^n.
define abel_center_seq(a: Nat -> Real, s: Real, x: Real, n: Nat) -> Real {
    (partial(a, n.suc) - s) * x.pow(n)
}

/// The absolute deviation |s_{n+1} - s| of the partial sums from s.
define abel_diff_abs(a: Nat -> Real, s: Real, n: Nat) -> Real {
    (partial(a, n.suc) - s).abs
}

/// Powers of a number in [0, 1] do not exceed one.
theorem pow_le_one(x: Real, n: Nat) {
    Real.0 <= x and x <= Real.1 implies x.pow(n) <= Real.1
} by {
    if Real.0 <= x and x <= Real.1 {
        pow_lte_base(x, Real.1, n)
        x.pow(n) <= Real.1.pow(n)
        one_pow[Real](n)
        Real.1.pow(n) = Real.1
        x.pow(n) <= Real.1
    }
}

/// Introduction rule for converges_to: the Weierstrass tail-bound form.
theorem converges_to_intro(q: Nat -> Real, a: Real) {
    (forall(eps: Real) { eps.is_positive implies exists(n: Nat) { tail_bound(q, a, n, eps) } })
    implies converges_to(q, a)
} by {
    if forall(eps: Real) { eps.is_positive implies exists(n: Nat) { tail_bound(q, a, n, eps) } } {
        // The definition of converges_to unfolds to exactly the hypothesis.
        forall(eps: Real) {
            if eps.is_positive {
                forall(e: Real) { e.is_positive implies exists(n: Nat) { tail_bound(q, a, n, e) } }
                exists(n: Nat) { tail_bound(q, a, n, eps) }
            }
        }
    }
}

/// The triangle inequality: |a + b| <= |a| + |b|.
theorem abs_add_le(a: Real, b: Real) {
    (a + b).abs <= a.abs + b.abs
} by {
    lte_abs(a)
    a <= a.abs
    lte_abs(b)
    b <= b.abs
    add_lte_add(a, a.abs, b, b.abs)
    a + b <= a.abs + b.abs
    lte_abs(-a)
    -a <= (-a).abs
    abs_neg(a)
    (-a).abs = a.abs
    -a <= a.abs
    lte_abs(-b)
    -b <= (-b).abs
    abs_neg(b)
    (-b).abs = b.abs
    -b <= b.abs
    add_lte_add(-a, a.abs, -b, b.abs)
    -a + -b <= a.abs + b.abs
    neg_distrib(a, b)
    -(a + b) = -a + -b
    -(a + b) <= a.abs + b.abs
    if (a + b).is_negative {
        (a + b).abs = -(a + b)
        lte_trans((a + b).abs, -(a + b), a.abs + b.abs)
        (a + b).abs <= a.abs + b.abs
    } else {
        (a + b).abs = a + b
        lte_trans((a + b).abs, a + b, a.abs + b.abs)
        (a + b).abs <= a.abs + b.abs
    }
}

// ---------------------------------------------------------------------------
// Finite Abel identities.
// ---------------------------------------------------------------------------

/// The finite geometric sum: Σ_{k<n} r^k · (1 - r) = 1 - r^n.
theorem geom_partial_sum_eq(r: Real, n: Nat) {
    partial(r.pow, n) * (Real.1 - r) = Real.1 - r.pow(n)
} by {
    define p(k: Nat) -> Bool {
        partial(r.pow, k) * (Real.1 - r) = Real.1 - r.pow(k)
    }
    // Base case: k = 0.
    partial_zero(r.pow)
    partial(r.pow, Nat.0) = Real.0
    Real.0 * (Real.1 - r) = Real.0
    pow_zero(r)
    r.pow(Nat.0) = Real.1
    Real.1 - Real.1 = Real.0
    p(Nat.0)
    // Inductive step.
    forall(k: Nat) {
        if p(k) {
            p(k) = (partial(r.pow, k) * (Real.1 - r) = Real.1 - r.pow(k))
            partial(r.pow, k) * (Real.1 - r) = Real.1 - r.pow(k)
            partial_suc(r.pow, k)
            partial(r.pow, k.suc) = partial(r.pow, k) + r.pow(k)
            partial(r.pow, k.suc) * (Real.1 - r) =
                (partial(r.pow, k) + r.pow(k)) * (Real.1 - r)
            (partial(r.pow, k) + r.pow(k)) * (Real.1 - r) =
                partial(r.pow, k) * (Real.1 - r) + r.pow(k) * (Real.1 - r)
            partial(r.pow, k) * (Real.1 - r) + r.pow(k) * (Real.1 - r) =
                (Real.1 - r.pow(k)) + r.pow(k) * (Real.1 - r)
            mul_sub_distrib_right(r.pow(k), Real.1, r)
            r.pow(k) * (Real.1 - r) = r.pow(k) * Real.1 - r.pow(k) * r
            r.pow(k) * Real.1 = r.pow(k)
            pow_suc(r, k)
            r.pow(k.suc) = r * r.pow(k)
            r.pow(k) * r = r.pow(k.suc)
            r.pow(k) * (Real.1 - r) = r.pow(k) - r.pow(k.suc)
            (Real.1 - r.pow(k)) + (r.pow(k) - r.pow(k.suc)) = Real.1 - r.pow(k.suc)
            partial(r.pow, k.suc) * (Real.1 - r) = Real.1 - r.pow(k.suc)
            p(k.suc)
        }
    }
    p(Nat.0) and forall(k: Nat) { p(k) implies p(k.suc) }
    Nat.induction(p)
    p(n)
}

/// The Abel sequence splits into the centered part and the geometric part.
theorem abel_seq_split(a: Nat -> Real, s: Real, x: Real, n: Nat) {
    abel_seq(a, x, n) = abel_center_seq(a, s, x, n) + s * x.pow(n)
} by {
    abel_seq(a, x, n) = partial(a, n.suc) * x.pow(n)
    abel_center_seq(a, s, x, n) = (partial(a, n.suc) - s) * x.pow(n)
    partial(a, n.suc) - s + s = partial(a, n.suc)
    (partial(a, n.suc) - s) * x.pow(n) + s * x.pow(n) =
        (partial(a, n.suc) - s + s) * x.pow(n)
    (partial(a, n.suc) - s + s) * x.pow(n) = partial(a, n.suc) * x.pow(n)
    abel_center_seq(a, s, x, n) + s * x.pow(n) =
        partial(a, n.suc) * x.pow(n)
    abel_seq(a, x, n) = abel_center_seq(a, s, x, n) + s * x.pow(n)
}

/// The finite Abel identity:
/// Σ_{k<n} a(k) x^k = (1-x) Σ_{k<n} s_{k+1} x^k + s_n x^n.
theorem abel_finite_identity(a: Nat -> Real, x: Real, n: Nat) {
    partial(power_series_term(a, x), n) =
        (Real.1 - x) * partial(abel_seq(a, x), n) + partial(a, n) * x.pow(n)
} by {
    define p(k: Nat) -> Bool {
        partial(power_series_term(a, x), k) =
            (Real.1 - x) * partial(abel_seq(a, x), k) + partial(a, k) * x.pow(k)
    }
    // Base case: k = 0.
    partial_zero(power_series_term(a, x))
    partial(power_series_term(a, x), Nat.0) = Real.0
    partial_zero(abel_seq(a, x))
    partial(abel_seq(a, x), Nat.0) = Real.0
    partial_zero(a)
    partial(a, Nat.0) = Real.0
    pow_zero(x)
    x.pow(Nat.0) = Real.1
    mul_zero_right(Real.1 - x)
    (Real.1 - x) * Real.0 = Real.0
    mul_zero_right(Real.0)
    Real.0 * Real.1 = Real.0
    add_zero_right(Real.0)
    Real.0 + Real.0 = Real.0
    p(Nat.0)
    // Inductive step.
    forall(k: Nat) {
        if p(k) {
            p(k) = (partial(power_series_term(a, x), k) =
                (Real.1 - x) * partial(abel_seq(a, x), k) + partial(a, k) * x.pow(k))
            partial(power_series_term(a, x), k) =
                (Real.1 - x) * partial(abel_seq(a, x), k) + partial(a, k) * x.pow(k)
            partial_suc(power_series_term(a, x), k)
            partial(power_series_term(a, x), k.suc) =
                partial(power_series_term(a, x), k) + power_series_term(a, x, k)
            power_series_term(a, x, k) = a(k) * x.pow(k)
            partial_suc(abel_seq(a, x), k)
            partial(abel_seq(a, x), k.suc) =
                partial(abel_seq(a, x), k) + abel_seq(a, x, k)
            abel_seq(a, x, k) = partial(a, k.suc) * x.pow(k)
            partial_suc(a, k)
            partial(a, k.suc) = partial(a, k) + a(k)
            // The step term is a(k) x^k.  First, with P = s_{k+1}:
            // (1 - x) P x^k + P x^{k+1} = P x^k.
            abel_seq(a, x, k) = partial(a, k.suc) * x.pow(k)
            pow_suc(x, k)
            x.pow(k.suc) = x * x.pow(k)
            mul_sub_distrib_left(Real.1, x, partial(a, k.suc) * x.pow(k))
            (Real.1 - x) * (partial(a, k.suc) * x.pow(k)) =
                Real.1 * (partial(a, k.suc) * x.pow(k)) - x * (partial(a, k.suc) * x.pow(k))
            Real.1 * (partial(a, k.suc) * x.pow(k)) = partial(a, k.suc) * x.pow(k)
            x * (partial(a, k.suc) * x.pow(k)) = partial(a, k.suc) * (x * x.pow(k))
            x * x.pow(k) = x.pow(k.suc)
            partial(a, k.suc) * (x * x.pow(k)) = partial(a, k.suc) * x.pow(k.suc)
            x * (partial(a, k.suc) * x.pow(k)) = partial(a, k.suc) * x.pow(k.suc)
            (Real.1 - x) * (partial(a, k.suc) * x.pow(k)) =
                partial(a, k.suc) * x.pow(k) - partial(a, k.suc) * x.pow(k.suc)
            (Real.1 - x) * (partial(a, k.suc) * x.pow(k)) + partial(a, k.suc) * x.pow(k.suc) =
                partial(a, k.suc) * x.pow(k)
            (Real.1 - x) * abel_seq(a, x, k) + partial(a, k.suc) * x.pow(k.suc) =
                partial(a, k.suc) * x.pow(k)
            // Subtract s_k x^k: the difference is (s_{k+1} - s_k) x^k = a(k) x^k.
            (Real.1 - x) * abel_seq(a, x, k) + partial(a, k.suc) * x.pow(k.suc) -
                partial(a, k) * x.pow(k) = partial(a, k.suc) * x.pow(k) - partial(a, k) * x.pow(k)
            sub_cancels(partial(a, k), a(k))
            partial(a, k) + a(k) - partial(a, k) = a(k)
            partial(a, k.suc) = partial(a, k) + a(k)
            partial(a, k.suc) - partial(a, k) = a(k)
            mul_sub_distrib_left(partial(a, k.suc), partial(a, k), x.pow(k))
            (partial(a, k.suc) - partial(a, k)) * x.pow(k) =
                partial(a, k.suc) * x.pow(k) - partial(a, k) * x.pow(k)
            (partial(a, k.suc) - partial(a, k)) * x.pow(k) = a(k) * x.pow(k)
            partial(a, k.suc) * x.pow(k) - partial(a, k) * x.pow(k) = a(k) * x.pow(k)
            (Real.1 - x) * abel_seq(a, x, k) + partial(a, k.suc) * x.pow(k.suc) -
                partial(a, k) * x.pow(k) = a(k) * x.pow(k)
            // RHS(k+1) = RHS(k) + a(k) x^k.
            partial_suc(abel_seq(a, x), k)
            partial(abel_seq(a, x), k.suc) = partial(abel_seq(a, x), k) + abel_seq(a, x, k)
            partial_suc(a, k)
            partial(a, k.suc) = partial(a, k) + a(k)
            pow_suc(x, k)
            x.pow(k.suc) = x * x.pow(k)
            mul_distrib_left(Real.1 - x, partial(abel_seq(a, x), k), abel_seq(a, x, k))
            (Real.1 - x) * (partial(abel_seq(a, x), k) + abel_seq(a, x, k)) =
                (Real.1 - x) * partial(abel_seq(a, x), k) + (Real.1 - x) * abel_seq(a, x, k)
            mul_distrib_left(partial(a, k), a(k), x.pow(k.suc))
            (partial(a, k) + a(k)) * x.pow(k.suc) =
                partial(a, k) * x.pow(k.suc) + a(k) * x.pow(k.suc)
            (Real.1 - x) * (partial(abel_seq(a, x), k) + abel_seq(a, x, k)) +
                (partial(a, k) + a(k)) * x.pow(k.suc) =
                (Real.1 - x) * partial(abel_seq(a, x), k) + (Real.1 - x) * abel_seq(a, x, k) +
                partial(a, k) * x.pow(k.suc) + a(k) * x.pow(k.suc)
            (Real.1 - x) * partial(abel_seq(a, x), k.suc) + partial(a, k.suc) * x.pow(k.suc) =
                (Real.1 - x) * partial(abel_seq(a, x), k) + (Real.1 - x) * abel_seq(a, x, k) +
                partial(a, k) * x.pow(k.suc) + a(k) * x.pow(k.suc)
            // The step-term identity: (1-x)abel(k) + s_{k+1} x^{k+1} = s_k x^k + a_k x^k.
            (Real.1 - x) * abel_seq(a, x, k) + partial(a, k.suc) * x.pow(k.suc) -
                partial(a, k) * x.pow(k) = a(k) * x.pow(k)
            (Real.1 - x) * abel_seq(a, x, k) + partial(a, k.suc) * x.pow(k.suc) =
                partial(a, k) * x.pow(k) + a(k) * x.pow(k)
            // Expand s_{k+1} x^{k+1} in the identity.
            partial(a, k.suc) * x.pow(k.suc) =
                partial(a, k) * x.pow(k.suc) + a(k) * x.pow(k.suc)
            (Real.1 - x) * abel_seq(a, x, k) +
                partial(a, k) * x.pow(k.suc) + a(k) * x.pow(k.suc) =
                partial(a, k) * x.pow(k) + a(k) * x.pow(k)
            // Regroup: X + Y + C = X + (Y + C).
            add_assoc((Real.1 - x) * partial(abel_seq(a, x), k),
                (Real.1 - x) * abel_seq(a, x, k), partial(a, k.suc) * x.pow(k.suc))
            (Real.1 - x) * partial(abel_seq(a, x), k) +
                ((Real.1 - x) * abel_seq(a, x, k) + partial(a, k.suc) * x.pow(k.suc)) =
                (Real.1 - x) * partial(abel_seq(a, x), k) +
                (Real.1 - x) * abel_seq(a, x, k) + partial(a, k.suc) * x.pow(k.suc)
            // Substitute the step-term identity into the bracket.
            (Real.1 - x) * abel_seq(a, x, k) + partial(a, k.suc) * x.pow(k.suc) =
                partial(a, k) * x.pow(k) + a(k) * x.pow(k)
            (Real.1 - x) * partial(abel_seq(a, x), k) +
                ((Real.1 - x) * abel_seq(a, x, k) + partial(a, k.suc) * x.pow(k.suc)) =
                (Real.1 - x) * partial(abel_seq(a, x), k) +
                (partial(a, k) * x.pow(k) + a(k) * x.pow(k))
            add_assoc((Real.1 - x) * partial(abel_seq(a, x), k),
                partial(a, k) * x.pow(k), a(k) * x.pow(k))
            (Real.1 - x) * partial(abel_seq(a, x), k) +
                (partial(a, k) * x.pow(k) + a(k) * x.pow(k)) =
                (Real.1 - x) * partial(abel_seq(a, x), k) +
                partial(a, k) * x.pow(k) + a(k) * x.pow(k)
            (Real.1 - x) * partial(abel_seq(a, x), k) +
                ((Real.1 - x) * abel_seq(a, x, k) + partial(a, k.suc) * x.pow(k.suc)) =
                (Real.1 - x) * partial(abel_seq(a, x), k) +
                partial(a, k) * x.pow(k) + a(k) * x.pow(k)
            (Real.1 - x) * partial(abel_seq(a, x), k.suc) + partial(a, k.suc) * x.pow(k.suc) =
                (Real.1 - x) * partial(abel_seq(a, x), k) + partial(a, k) * x.pow(k) + a(k) * x.pow(k)
            partial(power_series_term(a, x), k.suc) = partial(power_series_term(a, x), k) + a(k) * x.pow(k)
            partial(power_series_term(a, x), k.suc) =
                (Real.1 - x) * partial(abel_seq(a, x), k) + partial(a, k) * x.pow(k) + a(k) * x.pow(k)
            partial(power_series_term(a, x), k.suc) =
                (Real.1 - x) * partial(abel_seq(a, x), k.suc) + partial(a, k.suc) * x.pow(k.suc)
            p(k.suc)
        }
    }
    p(Nat.0) and forall(k: Nat) { p(k) implies p(k.suc) }
    Nat.induction(p)
    p(n)
}

// ---------------------------------------------------------------------------
// Convergence of the centered Abel series.
// ---------------------------------------------------------------------------

/// The centered Abel series converges absolutely for 0 <= x < 1.
theorem abel_center_abs_conv(a: Nat -> Real, s: Real, x: Real) {
    converges_to(partial(a), s) and Real.0 <= x and x < Real.1
    implies absolutely_converges(abel_center_seq(a, s, x))
} by {
    if converges_to(partial(a), s) and Real.0 <= x and x < Real.1 {
        // The shifted partial sums s_{n+1} converge to s, hence are bounded.
        converges_to_imp_converges(partial(a), s)
        converges(partial(a))
        tail_converges_to(partial(a), Nat.1)
        converges_to(tail(partial(a), Nat.1), limit(partial(a)))
        converges_to_unique(partial(a), s, limit(partial(a)))
        s = limit(partial(a))
        converges_to(tail(partial(a), Nat.1), s)
        converges_to_imp_bounded_seq(tail(partial(a), Nat.1), s)
        is_bounded_seq(tail(partial(a), Nat.1))
        is_bounded_seq(tail(partial(a), Nat.1)) = exists(bound: Real) {
            forall(n: Nat) {
                tail(partial(a), Nat.1)(n).abs < bound
            }
        }
        exists(bound: Real) {
            forall(n: Nat) {
                tail(partial(a), Nat.1)(n).abs < bound
            }
        }
        let bound: Real satisfy {
            forall(n: Nat) {
                tail(partial(a), Nat.1)(n).abs < bound
            }
        }
        tail(partial(a), Nat.1)(Nat.0).abs < bound
        abs_gte_zero(tail(partial(a), Nat.1)(Nat.0))
        Real.0 <= tail(partial(a), Nat.1)(Nat.0).abs
        lte_lt_trans(Real.0, tail(partial(a), Nat.1)(Nat.0).abs, bound)
        Real.0 < bound
        lt_imp_lte(Real.0, bound)
        Real.0 <= bound
        // |s_{n+1} - s| < bound + |s| for every n.
        abs_gte_zero(s)
        Real.0 <= s.abs
        add_lte_add(Real.0, bound, Real.0, s.abs)
        Real.0 + Real.0 <= bound + s.abs
        Real.0 + Real.0 = Real.0
        Real.0 <= bound + s.abs
        forall(n: Nat) {
            tail(partial(a), Nat.1)(n).abs < bound
            tail(partial(a), Nat.1)(n) = partial(a, n.suc)
            partial(a, n.suc).abs < bound
            // |s_{n+1} - s| <= |s_{n+1}| + |s| < bound + |s|.
            abs_add_le(partial(a, n.suc), -s)
            (partial(a, n.suc) + -s).abs <= partial(a, n.suc).abs + (-s).abs
            partial(a, n.suc) - s = partial(a, n.suc) + -s
            (partial(a, n.suc) - s).abs <= partial(a, n.suc).abs + (-s).abs
            abs_neg(s)
            (-s).abs = s.abs
            (partial(a, n.suc) - s).abs <= partial(a, n.suc).abs + s.abs
            lt_add_right(partial(a, n.suc).abs, bound, s.abs)
            partial(a, n.suc).abs + s.abs < bound + s.abs
            lte_lt_trans((partial(a, n.suc) - s).abs,
                partial(a, n.suc).abs + s.abs, bound + s.abs)
            (partial(a, n.suc) - s).abs < bound + s.abs
        }
        // |cc(k)| <= (bound + |s|) x^k: the geometric majorant.
        forall(k: Nat) {
            if Nat.0 <= k {
                (partial(a, k.suc) - s).abs < bound + s.abs
                lt_imp_lte((partial(a, k.suc) - s).abs, bound + s.abs)
                (partial(a, k.suc) - s).abs <= bound + s.abs
                // |cc(k)| = |s_{k+1} - s| x^k.
                abs_fn(abel_center_seq(a, s, x))(k) = abel_center_seq(a, s, x, k).abs
                abel_center_seq(a, s, x, k) = (partial(a, k.suc) - s) * x.pow(k)
                mul_abs(partial(a, k.suc) - s, x.pow(k))
                abel_center_seq(a, s, x, k).abs =
                    (partial(a, k.suc) - s).abs * x.pow(k).abs
                abs_pow_abs(x, k)
                x.pow(k).abs = x.abs.pow(k)
                abs_eq_self_of_nonneg(x)
                Real.0 <= x implies x.abs = x
                x.abs = x
                x.abs.pow(k) = x.pow(k)
                x.pow(k).abs = x.pow(k)
                abel_center_seq(a, s, x, k).abs =
                    (partial(a, k.suc) - s).abs * x.pow(k)
                abs_fn(abel_center_seq(a, s, x))(k) =
                    (partial(a, k.suc) - s).abs * x.pow(k)
                pow_nonneg(x, k)
                Real.0 <= x.pow(k)
                mul_le_mul_of_nonneg_right((partial(a, k.suc) - s).abs, bound + s.abs, x.pow(k))
                (partial(a, k.suc) - s).abs * x.pow(k) <= (bound + s.abs) * x.pow(k)
                abs_fn(abel_center_seq(a, s, x))(k) <= (bound + s.abs) * x.pow(k)
                geometric_majorant(bound + s.abs, x, k) = (bound + s.abs) * x.pow(k)
                abs_fn(abel_center_seq(a, s, x))(k) <= geometric_majorant(bound + s.abs, x, k)
            }
        }
        forall(k: Nat) {
            Nat.0 <= k implies abs_fn(abel_center_seq(a, s, x))(k) <= geometric_majorant(bound + s.abs, x, k)
        }
        eventually_abs_le_geometric_absolutely_converges(abel_center_seq(a, s, x), bound + s.abs, x, Nat.0)
        absolutely_converges(abel_center_seq(a, s, x))
    }
}

// ---------------------------------------------------------------------------
// The limit identity for the centered Abel series.
// ---------------------------------------------------------------------------

/// The partial sums of the Abel sequence split into the centered part and the
/// geometric part: Σ_{k<n} s_{k+1} x^k = Σ_{k<n} (s_{k+1} - s) x^k + s Σ_{k<n} x^k.
theorem abel_seq_partial_split(a: Nat -> Real, s: Real, x: Real, n: Nat) {
    partial(abel_seq(a, x), n) =
        partial(abel_center_seq(a, s, x), n) + s * partial(x.pow, n)
} by {
    forall(k: Nat) {
        if k < n {
            abel_seq_split(a, s, x, k)
            abel_seq(a, x, k) = abel_center_seq(a, s, x, k) + s * x.pow(k)
            add_seq(abel_center_seq(a, s, x), mul_seq(s, x.pow))(k) =
                abel_center_seq(a, s, x, k) + mul_seq(s, x.pow, k)
            mul_seq(s, x.pow, k) = s * x.pow(k)
            abel_center_seq(a, s, x, k) + s * x.pow(k) =
                add_seq(abel_center_seq(a, s, x), mul_seq(s, x.pow))(k)
        }
    }
    forall(k: Nat) {
        k < n implies abel_seq(a, x, k) = add_seq(abel_center_seq(a, s, x), mul_seq(s, x.pow))(k)
    }
    partial_pointwise_eq(abel_seq(a, x), add_seq(abel_center_seq(a, s, x), mul_seq(s, x.pow)), n)
    partial(abel_seq(a, x), n) =
        partial(add_seq(abel_center_seq(a, s, x), mul_seq(s, x.pow)), n)
    partial_add_seq_comm(abel_center_seq(a, s, x), mul_seq(s, x.pow))
    partial(add_seq(abel_center_seq(a, s, x), mul_seq(s, x.pow))) =
        add_seq(partial(abel_center_seq(a, s, x)), partial(mul_seq(s, x.pow)))
    partial(add_seq(abel_center_seq(a, s, x), mul_seq(s, x.pow)), n) =
        add_seq(partial(abel_center_seq(a, s, x)), partial(mul_seq(s, x.pow)), n)
    partial(abel_seq(a, x), n) =
        add_seq(partial(abel_center_seq(a, s, x)), partial(mul_seq(s, x.pow)), n)
    partial_mul_seq_comm(s, x.pow)
    partial(mul_seq(s, x.pow)) = mul_seq(s, partial(x.pow))
    partial(mul_seq(s, x.pow), n) = mul_seq(s, partial(x.pow), n)
    mul_seq(s, partial(x.pow), n) = s * partial(x.pow, n)
    partial(mul_seq(s, x.pow), n) = s * partial(x.pow, n)
    add_seq(partial(abel_center_seq(a, s, x)), partial(mul_seq(s, x.pow)), n) =
        partial(abel_center_seq(a, s, x), n) + partial(mul_seq(s, x.pow), n)
    partial(abel_seq(a, x), n) =
        partial(abel_center_seq(a, s, x), n) + partial(mul_seq(s, x.pow), n)
    partial(abel_seq(a, x), n) =
        partial(abel_center_seq(a, s, x), n) + s * partial(x.pow, n)
}

/// The power series at x converges for 0 <= x < 1 when the series converges
/// at 1.
theorem abel_ps_conv(a: Nat -> Real, x: Real) {
    converges(partial(a)) and Real.0 <= x and x < Real.1
    implies converges(partial(power_series_term(a, x)))
} by {
    if converges(partial(a)) and Real.0 <= x and x < Real.1 {
        forall(n: Nat) {
            power_series_term(a, Real.1, n) = a(n) * Real.1.pow(n)
            one_pow[Real](n)
            Real.1.pow(n) = Real.1
            a(n) * Real.1 = a(n)
            power_series_term(a, Real.1, n) = a(n)
        }
        converges_pointwise_eq(power_series_term(a, Real.1), a)
        converges(partial(power_series_term(a, Real.1)))
        zero_is_different_than_one
        Real.1 != Real.0
        abs_eq_self_of_nonneg(x)
        Real.0 <= x implies x.abs = x
        x.abs = x
        lt_imp_lte(x, Real.1)
        x <= Real.1
        x.abs <= Real.1
        power_series_abs_conv_inside_radius(a, x, Real.1)
        absolutely_converges(power_series_term(a, x))
        absolutely_converges_imp_converges(power_series_term(a, x))
        converges(partial(power_series_term(a, x)))
    }
}

/// ps_sum(a, x) = limit(partial(power_series_term(a, x))).
theorem ps_sum_eq_power_series_limit(a: Nat -> Real, x: Real) {
    converges(partial(power_series_term(a, x)))
    implies ps_sum(a, x) = limit(partial(power_series_term(a, x)))
} by {
    if converges(partial(power_series_term(a, x))) {
        forall(n: Nat) {
            ps_term_eq_power_series_term(a, x, n)
            ps_term(a, x, n) = power_series_term(a, x, n)
        }
        converges_pointwise_eq(partial(ps_term(a, x)), partial(power_series_term(a, x)))
        converges(partial(ps_term(a, x)))
        limit_pointwise_eq(partial(ps_term(a, x)), partial(power_series_term(a, x)))
        limit(partial(power_series_term(a, x))) = limit(partial(ps_term(a, x)))
        ps_sum(a, x) = limit(partial(ps_term(a, x)))
        ps_sum(a, x) = limit(partial(power_series_term(a, x)))
    }
}

/// The centered Abel series converges for 0 <= x < 1.
theorem abel_center_converges(a: Nat -> Real, s: Real, x: Real) {
    converges_to(partial(a), s) and Real.0 <= x and x < Real.1
    implies converges(partial(abel_center_seq(a, s, x)))
} by {
    if converges_to(partial(a), s) and Real.0 <= x and x < Real.1 {
        abel_center_abs_conv(a, s, x)
        absolutely_converges(abel_center_seq(a, s, x))
        absolutely_converges_imp_converges(abel_center_seq(a, s, x))
        converges(partial(abel_center_seq(a, s, x)))
    }
}

/// The Abel series converges for 0 <= x < 1.
theorem abel_seq_converges(a: Nat -> Real, s: Real, x: Real) {
    converges_to(partial(a), s) and Real.0 <= x and x < Real.1
    implies converges(partial(abel_seq(a, x)))
} by {
    if converges_to(partial(a), s) and Real.0 <= x and x < Real.1 {
        converges_to_imp_converges(partial(a), s)
        converges(partial(a))
        abel_center_converges(a, s, x)
        converges(partial(abel_center_seq(a, s, x)))
        abs_eq_self_of_nonneg(x)
        Real.0 <= x implies x.abs = x
        x.abs = x
        x.abs < Real.1
        geom_converges(x)
        converges(partial(x.pow))
        partial_mul_seq_comm(s, x.pow)
        partial(mul_seq(s, x.pow)) = mul_seq(s, partial(x.pow))
        mul_seq_converges_to(s, partial(x.pow))
        converges_to(mul_seq(s, partial(x.pow)), s * limit(partial(x.pow)))
        converges_to_imp_converges(mul_seq(s, partial(x.pow)), s * limit(partial(x.pow)))
        converges(mul_seq(s, partial(x.pow)))
        converges_pointwise_eq(partial(mul_seq(s, x.pow)), mul_seq(s, partial(x.pow)))
        converges(partial(mul_seq(s, x.pow)))
        add_seq_converges(partial(abel_center_seq(a, s, x)), partial(mul_seq(s, x.pow)))
        converges(add_seq(partial(abel_center_seq(a, s, x)), partial(mul_seq(s, x.pow))))
        forall(n: Nat) {
            abel_seq_partial_split(a, s, x, n)
            partial(abel_seq(a, x), n) =
                partial(abel_center_seq(a, s, x), n) + s * partial(x.pow, n)
            add_seq(partial(abel_center_seq(a, s, x)), partial(mul_seq(s, x.pow)))(n) =
                partial(abel_center_seq(a, s, x), n) + partial(mul_seq(s, x.pow), n)
            partial(mul_seq(s, x.pow), n) = s * partial(x.pow, n)
            partial(abel_seq(a, x), n) =
                add_seq(partial(abel_center_seq(a, s, x)), partial(mul_seq(s, x.pow)))(n)
        }
        converges_pointwise_eq(partial(abel_seq(a, x)),
            add_seq(partial(abel_center_seq(a, s, x)), partial(mul_seq(s, x.pow))))
        converges(partial(abel_seq(a, x)))
    }
}

/// The limit identity: ps_sum(a, x) - s = (1 - x) · lim Σ (s_{n+1} - s) x^n.
theorem abel_limit_center_identity(a: Nat -> Real, s: Real, x: Real) {
    converges_to(partial(a), s) and Real.0 <= x and x < Real.1
    implies ps_sum(a, x) - s =
        (Real.1 - x) * limit(partial(abel_center_seq(a, s, x)))
} by {
    if converges_to(partial(a), s) and Real.0 <= x and x < Real.1 {
        converges_to_imp_converges(partial(a), s)
        converges(partial(a))
        abel_ps_conv(a, x)
        converges(partial(power_series_term(a, x)))
        ps_sum_eq_power_series_limit(a, x)
        ps_sum(a, x) = limit(partial(power_series_term(a, x)))
        abel_seq_converges(a, s, x)
        converges(partial(abel_seq(a, x)))
        abel_center_converges(a, s, x)
        converges(partial(abel_center_seq(a, s, x)))
        abs_eq_self_of_nonneg(x)
        Real.0 <= x implies x.abs = x
        x.abs = x
        x.abs < Real.1
        geom_converges(x)
        converges(partial(x.pow))
        // The RHS sequence of the finite identity converges to (1-x) L_abel.
        series_conv_imp_term_vanishes(x.pow)
        converges_to(x.pow, Real.0)
        limit_of_product(partial(a), x.pow, s, Real.0)
        converges_to(prod_seq(partial(a), x.pow), s * Real.0)
        mul_zero_right(s)
        s * Real.0 = Real.0
        converges_to(prod_seq(partial(a), x.pow), Real.0)
        mul_seq_converges_to(Real.1 - x, partial(abel_seq(a, x)))
        converges_to(mul_seq(Real.1 - x, partial(abel_seq(a, x))),
            (Real.1 - x) * limit(partial(abel_seq(a, x))))
        limit_add_seq(mul_seq(Real.1 - x, partial(abel_seq(a, x))), prod_seq(partial(a), x.pow))
        converges_to(add_seq(mul_seq(Real.1 - x, partial(abel_seq(a, x))), prod_seq(partial(a), x.pow)),
            (Real.1 - x) * limit(partial(abel_seq(a, x))) + Real.0)
        // The partial sums of the power series equal this sequence pointwise.
        forall(n: Nat) {
            abel_finite_identity(a, x, n)
            partial(power_series_term(a, x), n) =
                (Real.1 - x) * partial(abel_seq(a, x), n) + partial(a, n) * x.pow(n)
            add_seq(mul_seq(Real.1 - x, partial(abel_seq(a, x))), prod_seq(partial(a), x.pow))(n) =
                mul_seq(Real.1 - x, partial(abel_seq(a, x)), n) + prod_seq(partial(a), x.pow, n)
            mul_seq(Real.1 - x, partial(abel_seq(a, x)), n) =
                (Real.1 - x) * partial(abel_seq(a, x), n)
            prod_seq(partial(a), x.pow, n) = partial(a, n) * x.pow(n)
            partial(power_series_term(a, x), n) =
                add_seq(mul_seq(Real.1 - x, partial(abel_seq(a, x))), prod_seq(partial(a), x.pow))(n)
        }
        limit_pointwise_eq(partial(power_series_term(a, x)),
            add_seq(mul_seq(Real.1 - x, partial(abel_seq(a, x))), prod_seq(partial(a), x.pow)))
        limit(add_seq(mul_seq(Real.1 - x, partial(abel_seq(a, x))), prod_seq(partial(a), x.pow))) =
            limit(partial(power_series_term(a, x)))
        converges_to_imp_converges(add_seq(mul_seq(Real.1 - x, partial(abel_seq(a, x))), prod_seq(partial(a), x.pow)),
            (Real.1 - x) * limit(partial(abel_seq(a, x))) + Real.0)
        converges(add_seq(mul_seq(Real.1 - x, partial(abel_seq(a, x))), prod_seq(partial(a), x.pow)))
        converges_imp_converges_to(add_seq(mul_seq(Real.1 - x, partial(abel_seq(a, x))), prod_seq(partial(a), x.pow)))
        converges_to(add_seq(mul_seq(Real.1 - x, partial(abel_seq(a, x))), prod_seq(partial(a), x.pow)),
            limit(add_seq(mul_seq(Real.1 - x, partial(abel_seq(a, x))), prod_seq(partial(a), x.pow))))
        converges_to_unique(add_seq(mul_seq(Real.1 - x, partial(abel_seq(a, x))), prod_seq(partial(a), x.pow)),
            (Real.1 - x) * limit(partial(abel_seq(a, x))) + Real.0,
            limit(add_seq(mul_seq(Real.1 - x, partial(abel_seq(a, x))), prod_seq(partial(a), x.pow))))
        limit(add_seq(mul_seq(Real.1 - x, partial(abel_seq(a, x))), prod_seq(partial(a), x.pow))) =
            (Real.1 - x) * limit(partial(abel_seq(a, x))) + Real.0
        limit(partial(power_series_term(a, x))) =
            (Real.1 - x) * limit(partial(abel_seq(a, x))) + Real.0
        add_zero_right((Real.1 - x) * limit(partial(abel_seq(a, x))))
        (Real.1 - x) * limit(partial(abel_seq(a, x))) + Real.0 =
            (Real.1 - x) * limit(partial(abel_seq(a, x)))
        limit(partial(power_series_term(a, x))) =
            (Real.1 - x) * limit(partial(abel_seq(a, x)))
        ps_sum(a, x) = (Real.1 - x) * limit(partial(abel_seq(a, x)))
        // Relate the Abel limit to the centered limit:
        // limit(partial(abel_seq)) = limit(partial(cc)) + s / (1 - x).
        partial_mul_seq_comm(s, x.pow)
        partial(mul_seq(s, x.pow)) = mul_seq(s, partial(x.pow))
        mul_seq_converges_to(s, partial(x.pow))
        converges_to(mul_seq(s, partial(x.pow)), s * limit(partial(x.pow)))
        converges_to_imp_converges(mul_seq(s, partial(x.pow)), s * limit(partial(x.pow)))
        converges(mul_seq(s, partial(x.pow)))
        converges_pointwise_eq(partial(mul_seq(s, x.pow)), mul_seq(s, partial(x.pow)))
        converges(partial(mul_seq(s, x.pow)))
        limit_add_seq(partial(abel_center_seq(a, s, x)), partial(mul_seq(s, x.pow)))
        converges_to(add_seq(partial(abel_center_seq(a, s, x)), partial(mul_seq(s, x.pow))),
            limit(partial(abel_center_seq(a, s, x))) + limit(partial(mul_seq(s, x.pow))))
        forall(n: Nat) {
            abel_seq_partial_split(a, s, x, n)
            partial(abel_seq(a, x), n) =
                partial(abel_center_seq(a, s, x), n) + s * partial(x.pow, n)
            add_seq(partial(abel_center_seq(a, s, x)), partial(mul_seq(s, x.pow)))(n) =
                partial(abel_center_seq(a, s, x), n) + partial(mul_seq(s, x.pow), n)
            partial(mul_seq(s, x.pow), n) = s * partial(x.pow, n)
            partial(abel_seq(a, x), n) =
                add_seq(partial(abel_center_seq(a, s, x)), partial(mul_seq(s, x.pow)))(n)
        }
        limit_pointwise_eq(partial(abel_seq(a, x)),
            add_seq(partial(abel_center_seq(a, s, x)), partial(mul_seq(s, x.pow))))
        limit(add_seq(partial(abel_center_seq(a, s, x)), partial(mul_seq(s, x.pow)))) =
            limit(partial(abel_seq(a, x)))
        converges_to_imp_converges(add_seq(partial(abel_center_seq(a, s, x)), partial(mul_seq(s, x.pow))),
            limit(partial(abel_center_seq(a, s, x))) + limit(partial(mul_seq(s, x.pow))))
        converges(add_seq(partial(abel_center_seq(a, s, x)), partial(mul_seq(s, x.pow))))
        converges_imp_converges_to(add_seq(partial(abel_center_seq(a, s, x)), partial(mul_seq(s, x.pow))))
        converges_to(add_seq(partial(abel_center_seq(a, s, x)), partial(mul_seq(s, x.pow))),
            limit(add_seq(partial(abel_center_seq(a, s, x)), partial(mul_seq(s, x.pow)))))
        converges_to_unique(add_seq(partial(abel_center_seq(a, s, x)), partial(mul_seq(s, x.pow))),
            limit(partial(abel_center_seq(a, s, x))) + limit(partial(mul_seq(s, x.pow))),
            limit(add_seq(partial(abel_center_seq(a, s, x)), partial(mul_seq(s, x.pow)))))
        limit(partial(abel_center_seq(a, s, x))) + limit(partial(mul_seq(s, x.pow))) =
            limit(add_seq(partial(abel_center_seq(a, s, x)), partial(mul_seq(s, x.pow))))
        limit(partial(abel_seq(a, x))) =
            limit(partial(abel_center_seq(a, s, x))) + limit(partial(mul_seq(s, x.pow)))
        // limit(partial(mul_seq(s, x.pow))) = s / (1 - x).
        mul_seq_converges_to(s, partial(x.pow))
        converges_to(mul_seq(s, partial(x.pow)), s * limit(partial(x.pow)))
        converges_pointwise_eq(partial(mul_seq(s, x.pow)), mul_seq(s, partial(x.pow)))
        limit_pointwise_eq(partial(mul_seq(s, x.pow)), mul_seq(s, partial(x.pow)))
        limit(mul_seq(s, partial(x.pow))) = limit(partial(mul_seq(s, x.pow)))
        converges_imp_converges_to(mul_seq(s, partial(x.pow)))
        converges_to(mul_seq(s, partial(x.pow)), limit(mul_seq(s, partial(x.pow))))
        converges_to_unique(mul_seq(s, partial(x.pow)), s * limit(partial(x.pow)),
            limit(mul_seq(s, partial(x.pow))))
        s * limit(partial(x.pow)) = limit(mul_seq(s, partial(x.pow)))
        limit(partial(mul_seq(s, x.pow))) = s * limit(partial(x.pow))
        geom_series(x)
        limit(partial(x.pow)) = Real.1 / (Real.1 - x)
        limit(partial(mul_seq(s, x.pow))) = s * (Real.1 / (Real.1 - x))
        s * (Real.1 / (Real.1 - x)) = s / (Real.1 - x)
        limit(partial(mul_seq(s, x.pow))) = s / (Real.1 - x)
        limit(partial(abel_seq(a, x))) =
            limit(partial(abel_center_seq(a, s, x))) + s / (Real.1 - x)
        ps_sum(a, x) =
            (Real.1 - x) * (limit(partial(abel_center_seq(a, s, x))) + s / (Real.1 - x))
        mul_distrib_left(Real.1 - x, limit(partial(abel_center_seq(a, s, x))), s / (Real.1 - x))
        (Real.1 - x) * (limit(partial(abel_center_seq(a, s, x))) + s / (Real.1 - x)) =
            (Real.1 - x) * limit(partial(abel_center_seq(a, s, x))) +
            (Real.1 - x) * (s / (Real.1 - x))
        lt_imp_pos_sub(x, Real.1)
        Real.0 < Real.1 - x
        lt_iff_lte_and_ne[Real](Real.0, Real.1 - x)
        Real.0 < Real.1 - x = (Real.0 <= Real.1 - x and Real.0 != Real.1 - x)
        Real.0 != Real.1 - x
        Real.1 - x != Real.0
        mul_div_cancel(s, Real.1 - x)
        (Real.1 - x) * (s / (Real.1 - x)) = s
        ps_sum(a, x) =
            (Real.1 - x) * limit(partial(abel_center_seq(a, s, x))) + s
        ps_sum(a, x) - s =
            (Real.1 - x) * limit(partial(abel_center_seq(a, s, x)))
    }
}

// ---------------------------------------------------------------------------
// The Abel estimate and Abel's theorem.
// ---------------------------------------------------------------------------

/// The absolute value of a centered Abel term is at most |s_{k+1} - s|.
theorem abel_center_term_abs_le_diff(a: Nat -> Real, s: Real, x: Real, k: Nat) {
    Real.0 <= x and x <= Real.1
    implies abs_fn(abel_center_seq(a, s, x))(k) <= abel_diff_abs(a, s, k)
} by {
    if Real.0 <= x and x <= Real.1 {
        abs_fn(abel_center_seq(a, s, x))(k) = abel_center_seq(a, s, x, k).abs
        abel_center_seq(a, s, x, k) = (partial(a, k.suc) - s) * x.pow(k)
        mul_abs(partial(a, k.suc) - s, x.pow(k))
        abel_center_seq(a, s, x, k).abs =
            (partial(a, k.suc) - s).abs * x.pow(k).abs
        abs_pow_abs(x, k)
        x.pow(k).abs = x.abs.pow(k)
        abs_eq_self_of_nonneg(x)
        Real.0 <= x implies x.abs = x
        x.abs = x
        x.abs.pow(k) = x.pow(k)
        x.pow(k).abs = x.pow(k)
        abel_center_seq(a, s, x, k).abs =
            (partial(a, k.suc) - s).abs * x.pow(k)
        pow_le_one(x, k)
        x.pow(k) <= Real.1
        pow_nonneg(x, k)
        Real.0 <= x.pow(k)
        mul_le_mul_of_nonneg_left(x.pow(k), Real.1, (partial(a, k.suc) - s).abs)
        x.pow(k) <= Real.1 and Real.0 <= (partial(a, k.suc) - s).abs
        (partial(a, k.suc) - s).abs * x.pow(k) <= (partial(a, k.suc) - s).abs * Real.1
        (partial(a, k.suc) - s).abs * Real.1 = (partial(a, k.suc) - s).abs
        (partial(a, k.suc) - s).abs * x.pow(k) <= (partial(a, k.suc) - s).abs
        abel_center_seq(a, s, x, k).abs <= (partial(a, k.suc) - s).abs
        abel_diff_abs(a, s, k) = (partial(a, k.suc) - s).abs
        abel_center_seq(a, s, x, k).abs <= abel_diff_abs(a, s, k)
        abs_fn(abel_center_seq(a, s, x))(k) <= abel_diff_abs(a, s, k)
    }
}

/// The tail of the centered Abel series from n0 on is bounded in the limit by
/// eps x^n0 / (1 - x) when the deviations are below eps from n0 on.
theorem abel_tail_limit_bound(a: Nat -> Real, s: Real, x: Real, n0: Nat, eps: Real) {
    converges_to(partial(a), s) and Real.0 <= x and x < Real.1
    and (forall(k: Nat) { n0 <= k implies abel_diff_abs(a, s, k) < eps })
    implies limit(partial(tail(abel_center_seq(a, s, x), n0))).abs <= eps * x.pow(n0) / (Real.1 - x)
} by {
    if converges_to(partial(a), s) and Real.0 <= x and x < Real.1
        and (forall(k: Nat) { n0 <= k implies abel_diff_abs(a, s, k) < eps }) {
        abel_center_converges(a, s, x)
        converges(partial(abel_center_seq(a, s, x)))
        converges_to_imp_converges(partial(a), s)
        converges(partial(a))
        tail_partial_converges(abel_center_seq(a, s, x), n0)
        converges_to(partial(tail(abel_center_seq(a, s, x), n0)),
            limit(partial(abel_center_seq(a, s, x))) - partial(abel_center_seq(a, s, x), n0))
        converges_to_imp_converges(partial(tail(abel_center_seq(a, s, x), n0)),
            limit(partial(abel_center_seq(a, s, x))) - partial(abel_center_seq(a, s, x), n0))
        converges(partial(tail(abel_center_seq(a, s, x), n0)))
        // For every i, |tail(cc, n0)(i)| <= eps x^n0 x^i.
        forall(i: Nat) {
            abs_fn(tail(abel_center_seq(a, s, x), n0))(i) =
                tail(abel_center_seq(a, s, x), n0)(i).abs
            tail(abel_center_seq(a, s, x), n0)(i) = abel_center_seq(a, s, x, n0 + i)
            abel_center_seq(a, s, x, n0 + i) =
                (partial(a, (n0 + i).suc) - s) * x.pow(n0 + i)
            mul_abs(partial(a, (n0 + i).suc) - s, x.pow(n0 + i))
            tail(abel_center_seq(a, s, x), n0)(i).abs =
                (partial(a, (n0 + i).suc) - s).abs * x.pow(n0 + i).abs
            abs_pow_abs(x, n0 + i)
            x.pow(n0 + i).abs = x.abs.pow(n0 + i)
            abs_eq_self_of_nonneg(x)
            Real.0 <= x implies x.abs = x
            x.abs = x
            x.abs.pow(n0 + i) = x.pow(n0 + i)
            x.pow(n0 + i).abs = x.pow(n0 + i)
            tail(abel_center_seq(a, s, x), n0)(i).abs =
                (partial(a, (n0 + i).suc) - s).abs * x.pow(n0 + i)
            abs_fn(tail(abel_center_seq(a, s, x), n0))(i) =
                (partial(a, (n0 + i).suc) - s).abs * x.pow(n0 + i)
            // n0 <= n0 + i, so the deviation is below eps.
            lte_add_left(n0, Nat.0, i)
            Nat.0 <= i
            n0 + Nat.0 <= n0 + i
            n0 + Nat.0 = n0
            n0 <= n0 + i
            forall(k: Nat) { n0 <= k implies abel_diff_abs(a, s, k) < eps }
            abel_diff_abs(a, s, n0 + i) < eps
            abel_diff_abs(a, s, n0 + i) = (partial(a, (n0 + i).suc) - s).abs
            (partial(a, (n0 + i).suc) - s).abs < eps
            lt_imp_lte((partial(a, (n0 + i).suc) - s).abs, eps)
            (partial(a, (n0 + i).suc) - s).abs <= eps
            pow_add(x, n0, i)
            x.pow(n0 + i) = x.pow(n0) * x.pow(i)
            pow_nonneg(x, i)
            Real.0 <= x.pow(i)
            mul_le_mul_of_nonneg_right((partial(a, (n0 + i).suc) - s).abs, eps, x.pow(i))
            (partial(a, (n0 + i).suc) - s).abs * x.pow(i) <= eps * x.pow(i)
            pow_nonneg(x, n0)
            Real.0 <= x.pow(n0)
            mul_le_mul_of_nonneg_left(
                (partial(a, (n0 + i).suc) - s).abs * x.pow(i), eps * x.pow(i), x.pow(n0))
            x.pow(n0) * ((partial(a, (n0 + i).suc) - s).abs * x.pow(i)) <= x.pow(n0) * (eps * x.pow(i))
            real_mul_comm(x.pow(n0), (partial(a, (n0 + i).suc) - s).abs)
            x.pow(n0) * ((partial(a, (n0 + i).suc) - s).abs * x.pow(i)) =
                (partial(a, (n0 + i).suc) - s).abs * x.pow(n0) * x.pow(i)
            real_mul_comm(x.pow(n0), eps)
            x.pow(n0) * (eps * x.pow(i)) = eps * x.pow(n0) * x.pow(i)
            (partial(a, (n0 + i).suc) - s).abs * x.pow(n0) * x.pow(i) <= eps * x.pow(n0) * x.pow(i)
            (partial(a, (n0 + i).suc) - s).abs * x.pow(n0 + i) <= eps * x.pow(n0) * x.pow(i)
            abs_fn(tail(abel_center_seq(a, s, x), n0))(i) <= eps * x.pow(n0) * x.pow(i)
            mul_seq(eps * x.pow(n0), x.pow)(i) = eps * x.pow(n0) * x.pow(i)
            abs_fn(tail(abel_center_seq(a, s, x), n0))(i) <= mul_seq(eps * x.pow(n0), x.pow)(i)
        }
        seq_lte(abs_fn(tail(abel_center_seq(a, s, x), n0)), mul_seq(eps * x.pow(n0), x.pow))
        partial_seq_lte(abs_fn(tail(abel_center_seq(a, s, x), n0)), mul_seq(eps * x.pow(n0), x.pow))
        seq_lte(partial(abs_fn(tail(abel_center_seq(a, s, x), n0))),
            partial(mul_seq(eps * x.pow(n0), x.pow)))
        // The partial sums of the tail are bounded by eps x^n0 / (1 - x).
        forall(m: Nat) {
            seq_lte(partial(abs_fn(tail(abel_center_seq(a, s, x), n0))),
                partial(mul_seq(eps * x.pow(n0), x.pow)))
            partial(abs_fn(tail(abel_center_seq(a, s, x), n0)))(m) <= partial(mul_seq(eps * x.pow(n0), x.pow))(m)
            partial(abs_fn(tail(abel_center_seq(a, s, x), n0)), m) <= partial(mul_seq(eps * x.pow(n0), x.pow), m)
            partial_mul_seq_comm(eps * x.pow(n0), x.pow)
            partial(mul_seq(eps * x.pow(n0), x.pow)) =
                mul_seq(eps * x.pow(n0), partial(x.pow))
            partial(mul_seq(eps * x.pow(n0), x.pow), m) =
                mul_seq(eps * x.pow(n0), partial(x.pow), m)
            mul_seq(eps * x.pow(n0), partial(x.pow), m) =
                eps * x.pow(n0) * partial(x.pow, m)
            partial(abs_fn(tail(abel_center_seq(a, s, x), n0)), m) <= eps * x.pow(n0) * partial(x.pow, m)
            // partial(x.pow, m) <= 1 / (1 - x).
            lt_imp_pos_sub(x, Real.1)
            Real.0 < Real.1 - x
            pos_gt_zero(Real.1 - x)
            Real.1 - x > Real.0
            pos_geom_indirect_upper_bound(x, m)
            partial(x.pow, m) * (Real.1 - x) <= Real.1
            div_le_of_mul_le(partial(x.pow, m), Real.1 - x, Real.1)
            partial(x.pow, m) <= Real.1 / (Real.1 - x)
            // Multiply by eps x^n0 >= 0 (eps > 0 since |d_{n0}| < eps).
            n0 <= n0
            forall(k: Nat) { n0 <= k implies abel_diff_abs(a, s, k) < eps }
            abel_diff_abs(a, s, n0) < eps
            abs_gte_zero(partial(a, n0.suc) - s)
            Real.0 <= (partial(a, n0.suc) - s).abs
            abel_diff_abs(a, s, n0) = (partial(a, n0.suc) - s).abs
            Real.0 <= abel_diff_abs(a, s, n0)
            lte_lt_trans(Real.0, abel_diff_abs(a, s, n0), eps)
            Real.0 < eps
            lt_imp_lte(Real.0, eps)
            Real.0 <= eps
            eps >= Real.0
            pow_nonneg(x, n0)
            Real.0 <= x.pow(n0)
            x.pow(n0) >= Real.0
            eps >= Real.0 and x.pow(n0) >= Real.0
            mul_nonneg(eps, x.pow(n0))
            eps * x.pow(n0) >= Real.0
            Real.0 <= eps * x.pow(n0)
            mul_le_mul_of_nonneg_left(partial(x.pow, m), Real.1 / (Real.1 - x), eps * x.pow(n0))
            eps * x.pow(n0) * partial(x.pow, m) <= eps * x.pow(n0) * (Real.1 / (Real.1 - x))
            lte_trans(partial(abs_fn(tail(abel_center_seq(a, s, x), n0)), m),
                eps * x.pow(n0) * partial(x.pow, m),
                eps * x.pow(n0) * (Real.1 / (Real.1 - x)))
            partial(abs_fn(tail(abel_center_seq(a, s, x), n0)), m) <= eps * x.pow(n0) * (Real.1 / (Real.1 - x))
            eps * x.pow(n0) * (Real.1 / (Real.1 - x)) =
                eps * x.pow(n0) / (Real.1 - x)
            partial(abs_fn(tail(abel_center_seq(a, s, x), n0)), m) <= eps * x.pow(n0) / (Real.1 - x)
            partial_abs_le_partial_abs_fn(tail(abel_center_seq(a, s, x), n0), m)
            partial(tail(abel_center_seq(a, s, x), n0), m).abs <= partial(abs_fn(tail(abel_center_seq(a, s, x), n0)), m)
            lte_trans(partial(tail(abel_center_seq(a, s, x), n0), m).abs,
                partial(abs_fn(tail(abel_center_seq(a, s, x), n0)), m),
                eps * x.pow(n0) / (Real.1 - x))
            partial(tail(abel_center_seq(a, s, x), n0), m).abs <= eps * x.pow(n0) / (Real.1 - x)
            abs_seq(partial(tail(abel_center_seq(a, s, x), n0)))(m) =
                partial(tail(abel_center_seq(a, s, x), n0), m).abs
            abs_seq(partial(tail(abel_center_seq(a, s, x), n0)))(m) <= eps * x.pow(n0) / (Real.1 - x)
        }
        forall(i: Nat) {
            if Nat.0 <= i {
                abs_seq(partial(tail(abel_center_seq(a, s, x), n0)))(i) <= eps * x.pow(n0) / (Real.1 - x)
            }
        }
        exists(n: Nat) {
            forall(i: Nat) {
                n <= i implies abs_seq(partial(tail(abel_center_seq(a, s, x), n0)))(i) <= eps * x.pow(n0) / (Real.1 - x)
            }
        }
        limit_abs_seq(partial(tail(abel_center_seq(a, s, x), n0)))
        converges_to(abs_seq(partial(tail(abel_center_seq(a, s, x), n0))),
            limit(partial(tail(abel_center_seq(a, s, x), n0))).abs)
        converges_to_imp_converges(abs_seq(partial(tail(abel_center_seq(a, s, x), n0))),
            limit(partial(tail(abel_center_seq(a, s, x), n0))).abs)
        converges(abs_seq(partial(tail(abel_center_seq(a, s, x), n0))))
        eventual_ub(abs_seq(partial(tail(abel_center_seq(a, s, x), n0))),
            eps * x.pow(n0) / (Real.1 - x))
        ub_imp_limit_lte(abs_seq(partial(tail(abel_center_seq(a, s, x), n0))),
            eps * x.pow(n0) / (Real.1 - x))
        limit(abs_seq(partial(tail(abel_center_seq(a, s, x), n0)))) <= eps * x.pow(n0) / (Real.1 - x)
        converges_imp_converges_to(abs_seq(partial(tail(abel_center_seq(a, s, x), n0))))
        converges_to(abs_seq(partial(tail(abel_center_seq(a, s, x), n0))),
            limit(abs_seq(partial(tail(abel_center_seq(a, s, x), n0)))))
        converges_to_unique(abs_seq(partial(tail(abel_center_seq(a, s, x), n0))),
            limit(partial(tail(abel_center_seq(a, s, x), n0))).abs,
            limit(abs_seq(partial(tail(abel_center_seq(a, s, x), n0)))))
        limit(partial(tail(abel_center_seq(a, s, x), n0))).abs =
            limit(abs_seq(partial(tail(abel_center_seq(a, s, x), n0))))
        limit(partial(tail(abel_center_seq(a, s, x), n0))).abs <= eps * x.pow(n0) / (Real.1 - x)
    }
}

/// The Abel estimate: if the deviations |s_{k+1} - s| are below eps from n0 on,
/// then |ps_sum(a, x) - s| <= (1 - x) Σ_{k<n0} |s_{k+1} - s| + eps x^n0.
theorem abel_estimate(a: Nat -> Real, s: Real, x: Real, n0: Nat, eps: Real) {
    converges_to(partial(a), s) and Real.0 <= x and x < Real.1
    and (forall(k: Nat) { n0 <= k implies abel_diff_abs(a, s, k) < eps })
    implies (ps_sum(a, x) - s).abs <= (Real.1 - x) * partial(abel_diff_abs(a, s), n0) + eps * x.pow(n0)
} by {
    if converges_to(partial(a), s) and Real.0 <= x and x < Real.1
        and (forall(k: Nat) { n0 <= k implies abel_diff_abs(a, s, k) < eps }) {
        abel_limit_center_identity(a, s, x)
        ps_sum(a, x) - s =
            (Real.1 - x) * limit(partial(abel_center_seq(a, s, x)))
        // 1 - x >= 0, so |(1-x) L| = (1-x) |L|.
        lt_imp_pos_sub(x, Real.1)
        Real.0 < Real.1 - x
        lt_imp_lte(Real.0, Real.1 - x)
        Real.0 <= Real.1 - x
        abs_eq_self_of_nonneg(Real.1 - x)
        Real.1 - x >= Real.0 implies (Real.1 - x).abs = Real.1 - x
        (Real.1 - x).abs = Real.1 - x
        mul_abs(Real.1 - x, limit(partial(abel_center_seq(a, s, x))))
        ((Real.1 - x) * limit(partial(abel_center_seq(a, s, x)))).abs =
            (Real.1 - x).abs * limit(partial(abel_center_seq(a, s, x))).abs
        (ps_sum(a, x) - s).abs =
            (Real.1 - x) * limit(partial(abel_center_seq(a, s, x))).abs
        // L = partial(cc, n0) + limit(partial(tail(cc, n0))).
        abel_center_converges(a, s, x)
        converges(partial(abel_center_seq(a, s, x)))
        partial_tail_decomp(abel_center_seq(a, s, x), n0)
        limit(partial(abel_center_seq(a, s, x))) =
            partial(abel_center_seq(a, s, x), n0) +
            limit(partial(tail(abel_center_seq(a, s, x), n0)))
        // |L| <= |partial(cc, n0)| + |limit(tail)|.
        abs_add_le(partial(abel_center_seq(a, s, x), n0),
            limit(partial(tail(abel_center_seq(a, s, x), n0))))
        (partial(abel_center_seq(a, s, x), n0) +
            limit(partial(tail(abel_center_seq(a, s, x), n0)))).abs <= (partial(abel_center_seq(a, s, x), n0)).abs +
            limit(partial(tail(abel_center_seq(a, s, x), n0))).abs
        limit(partial(abel_center_seq(a, s, x))) =
            partial(abel_center_seq(a, s, x), n0) +
            limit(partial(tail(abel_center_seq(a, s, x), n0)))
        (limit(partial(abel_center_seq(a, s, x)))).abs <= (partial(abel_center_seq(a, s, x), n0)).abs +
            limit(partial(tail(abel_center_seq(a, s, x), n0))).abs
        // |partial(cc, n0)| <= partial(abs_fn(cc), n0) <= partial(|d|, n0).
        partial_abs_le_partial_abs_fn(abel_center_seq(a, s, x), n0)
        partial(abel_center_seq(a, s, x), n0).abs <= partial(abs_fn(abel_center_seq(a, s, x)), n0)
        lt_imp_lte(x, Real.1)
        x <= Real.1
        forall(k: Nat) {
            abel_center_term_abs_le_diff(a, s, x, k)
            abs_fn(abel_center_seq(a, s, x))(k) <= abel_diff_abs(a, s, k)
        }
        seq_lte(abs_fn(abel_center_seq(a, s, x)), abel_diff_abs(a, s))
        partial_seq_lte(abs_fn(abel_center_seq(a, s, x)), abel_diff_abs(a, s))
        seq_lte(partial(abs_fn(abel_center_seq(a, s, x))), partial(abel_diff_abs(a, s)))
        partial(abs_fn(abel_center_seq(a, s, x)), n0) <= partial(abel_diff_abs(a, s), n0)
        lte_trans(partial(abel_center_seq(a, s, x), n0).abs,
            partial(abs_fn(abel_center_seq(a, s, x)), n0),
            partial(abel_diff_abs(a, s), n0))
        partial(abel_center_seq(a, s, x), n0).abs <= partial(abel_diff_abs(a, s), n0)
        // |limit(tail)| <= eps x^n0 / (1 - x).
        abel_tail_limit_bound(a, s, x, n0, eps)
        limit(partial(tail(abel_center_seq(a, s, x), n0))).abs <= eps * x.pow(n0) / (Real.1 - x)
        // Combine: |L| <= C + eps x^n0 / (1-x).
        partial(abel_center_seq(a, s, x), n0).abs +
            limit(partial(tail(abel_center_seq(a, s, x), n0))).abs <= partial(abel_diff_abs(a, s), n0) + eps * x.pow(n0) / (Real.1 - x)
        lte_trans((limit(partial(abel_center_seq(a, s, x)))).abs,
            partial(abel_center_seq(a, s, x), n0).abs +
            limit(partial(tail(abel_center_seq(a, s, x), n0))).abs,
            partial(abel_diff_abs(a, s), n0) + eps * x.pow(n0) / (Real.1 - x))
        (limit(partial(abel_center_seq(a, s, x)))).abs <= partial(abel_diff_abs(a, s), n0) + eps * x.pow(n0) / (Real.1 - x)
        // (1-x) |L| <= (1-x) C + eps x^n0.
        mul_le_mul_of_nonneg_right((limit(partial(abel_center_seq(a, s, x)))).abs,
            partial(abel_diff_abs(a, s), n0) + eps * x.pow(n0) / (Real.1 - x),
            Real.1 - x)
        (limit(partial(abel_center_seq(a, s, x)))).abs * (Real.1 - x) <= (partial(abel_diff_abs(a, s), n0) + eps * x.pow(n0) / (Real.1 - x)) * (Real.1 - x)
        real_mul_comm((limit(partial(abel_center_seq(a, s, x)))).abs, Real.1 - x)
        (Real.1 - x) * (limit(partial(abel_center_seq(a, s, x)))).abs =
            (limit(partial(abel_center_seq(a, s, x)))).abs * (Real.1 - x)
        (Real.1 - x) * (limit(partial(abel_center_seq(a, s, x)))).abs <= (partial(abel_diff_abs(a, s), n0) + eps * x.pow(n0) / (Real.1 - x)) * (Real.1 - x)
        mul_distrib_left(partial(abel_diff_abs(a, s), n0), eps * x.pow(n0) / (Real.1 - x), Real.1 - x)
        (partial(abel_diff_abs(a, s), n0) + eps * x.pow(n0) / (Real.1 - x)) * (Real.1 - x) =
            partial(abel_diff_abs(a, s), n0) * (Real.1 - x) +
            (eps * x.pow(n0) / (Real.1 - x)) * (Real.1 - x)
        real_mul_comm(partial(abel_diff_abs(a, s), n0), Real.1 - x)
        partial(abel_diff_abs(a, s), n0) * (Real.1 - x) =
            (Real.1 - x) * partial(abel_diff_abs(a, s), n0)
        real_mul_comm(eps * x.pow(n0) / (Real.1 - x), Real.1 - x)
        (eps * x.pow(n0) / (Real.1 - x)) * (Real.1 - x) =
            (Real.1 - x) * (eps * x.pow(n0) / (Real.1 - x))
        (Real.1 - x) * (limit(partial(abel_center_seq(a, s, x)))).abs <= (Real.1 - x) * partial(abel_diff_abs(a, s), n0) +
            (Real.1 - x) * (eps * x.pow(n0) / (Real.1 - x))
        Real.1 - x != Real.0
        mul_div_cancel(eps * x.pow(n0), Real.1 - x)
        (Real.1 - x) * (eps * x.pow(n0) / (Real.1 - x)) = eps * x.pow(n0)
        (Real.1 - x) * (limit(partial(abel_center_seq(a, s, x)))).abs <= (Real.1 - x) * partial(abel_diff_abs(a, s), n0) + eps * x.pow(n0)
        (ps_sum(a, x) - s).abs <= (Real.1 - x) * partial(abel_diff_abs(a, s), n0) + eps * x.pow(n0)
    }
}


/// Half of a positive real plus half of it is the whole.
theorem half_plus_half(eps: Real) {
    eps / two + eps / two = eps
} by {
    eps / two + eps / two = (eps / two) * Real.1 + (eps / two) * Real.1
    (eps / two) * Real.1 + (eps / two) * Real.1 = (eps / two) * (Real.1 + Real.1)
    Real.1 + Real.1 = two
    (eps / two) * (Real.1 + Real.1) = (eps / two) * two
    two != Real.0
    mul_div_cancel(eps, two)
    two * (eps / two) = eps
    real_mul_comm(two, eps / two)
    (eps / two) * two = two * (eps / two)
    (eps / two) * two = eps
    eps / two + eps / two = eps
}

/// The sequential Abel theorem, parameterized by the target limit s.
theorem abel_theorem_at(a: Nat -> Real, s: Real, q: Nat -> Real) {
    converges_to(partial(a), s)
    and (forall(n: Nat) { Real.0 <= q(n) and q(n) < Real.1 })
    and converges_to(q, Real.1)
    implies converges_to(compose(ps_sum(a), q), s)
} by {
    if converges_to(partial(a), s)
        and (forall(n: Nat) { Real.0 <= q(n) and q(n) < Real.1 })
        and converges_to(q, Real.1) {
        converges_to_imp_converges(partial(a), s)
        converges(partial(a))
        forall(eps: Real) {
            if eps.is_positive {
                // eps2 = eps / 2 is positive.
                pos_gt_zero(eps)
                eps > Real.0
                two_positive
                two.is_positive
                pos_gt_zero(two)
                two > Real.0
                div_pos_of_pos_pos(eps, two)
                eps / two > Real.0
                gt_zero_imp_pos(eps / two)
                (eps / two).is_positive
                // The deviations are eventually below eps/2.
                converges_to(partial(a), s) = forall(e: Real) {
                    e.is_positive implies exists(n: Nat) {
                        tail_bound(partial(a), s, n, e)
                    }
                }
                (eps / two).is_positive implies exists(n: Nat) {
                    tail_bound(partial(a), s, n, eps / two)
                }
                exists(n: Nat) {
                    tail_bound(partial(a), s, n, eps / two)
                }
                let n1: Nat satisfy {
                    tail_bound(partial(a), s, n1, eps / two)
                }
                forall(k: Nat) {
                    if n1 <= k {
                        lt_suc(k)
                        k < k.suc
                        lt_imp_lte(k, k.suc)
                        k <= k.suc
                        lte_trans(n1, k, k.suc)
                        n1 <= k.suc
                        tail_bound_implies_is_close(partial(a), s, n1, eps / two, k.suc)
                        partial(a, k.suc).is_close(s, eps / two)
                        (partial(a, k.suc) - s).abs < eps / two
                        abel_diff_abs(a, s, k) = (partial(a, k.suc) - s).abs
                        abel_diff_abs(a, s, k) < eps / two
                    }
                }
                // The head c = Σ_{k<n1} |d_k|.
                abs_gte_zero(partial(abel_diff_abs(a, s), n1))
                Real.0 <= partial(abel_diff_abs(a, s), n1).abs
                Real.1.is_positive
                pos_gt_zero(Real.1)
                Real.1 > Real.0
                Real.0 < Real.1
                lt_add_pos(partial(abel_diff_abs(a, s), n1).abs, Real.1)
                partial(abel_diff_abs(a, s), n1).abs < partial(abel_diff_abs(a, s), n1).abs + Real.1
                lte_lt_trans(Real.0, Real.1,
                    partial(abel_diff_abs(a, s), n1).abs + Real.1)
                Real.0 < partial(abel_diff_abs(a, s), n1).abs + Real.1
                div_pos_of_pos_pos(eps / two, partial(abel_diff_abs(a, s), n1).abs + Real.1)
                (eps / two) / (partial(abel_diff_abs(a, s), n1).abs + Real.1) > Real.0
                gt_zero_imp_pos((eps / two) / (partial(abel_diff_abs(a, s), n1).abs + Real.1))
                ((eps / two) / (partial(abel_diff_abs(a, s), n1).abs + Real.1)).is_positive
                // q(n) is eventually within delta of 1.
                converges_to(q, Real.1) = forall(e: Real) {
                    e.is_positive implies exists(n: Nat) {
                        tail_bound(q, Real.1, n, e)
                    }
                }
                ((eps / two) / (partial(abel_diff_abs(a, s), n1).abs + Real.1)).is_positive
                exists(n: Nat) {
                    tail_bound(q, Real.1, n, (eps / two) / (partial(abel_diff_abs(a, s), n1).abs + Real.1))
                }
                let m: Nat satisfy {
                    tail_bound(q, Real.1, m, (eps / two) / (partial(abel_diff_abs(a, s), n1).abs + Real.1))
                }
                // Combine the two eventual bounds.
                let n2: Nat satisfy {
                    n1 <= n2 and m <= n2
                }
                forall(i: Nat) {
                    if n2 <= i {
                        // The estimate at x = q(i) with eps/2 and n1.
                        forall(n: Nat) { Real.0 <= q(n) and q(n) < Real.1 }
                        Real.0 <= q(i)
                        q(i) < Real.1
                        forall(k: Nat) {
                            if n1 <= k {
                                abel_diff_abs(a, s, k) < eps / two
                            }
                        }
                        abel_estimate(a, s, q(i), n1, eps / two)
                        (ps_sum(a, q(i)) - s).abs <= (Real.1 - q(i)) * partial(abel_diff_abs(a, s), n1) +
                            (eps / two) * q(i).pow(n1)
                        // (1 - q(i)) c <= (1 - q(i)) (c.abs + 1).
                        lte_abs(partial(abel_diff_abs(a, s), n1))
                        partial(abel_diff_abs(a, s), n1) <= partial(abel_diff_abs(a, s), n1).abs
                        lt_add_pos(partial(abel_diff_abs(a, s), n1).abs, Real.1)
                        partial(abel_diff_abs(a, s), n1).abs < partial(abel_diff_abs(a, s), n1).abs + Real.1
                        lt_imp_lte(partial(abel_diff_abs(a, s), n1).abs,
                            partial(abel_diff_abs(a, s), n1).abs + Real.1)
                        partial(abel_diff_abs(a, s), n1).abs <= partial(abel_diff_abs(a, s), n1).abs + Real.1
                        lte_trans(partial(abel_diff_abs(a, s), n1),
                            partial(abel_diff_abs(a, s), n1).abs,
                            partial(abel_diff_abs(a, s), n1).abs + Real.1)
                        partial(abel_diff_abs(a, s), n1) <= partial(abel_diff_abs(a, s), n1).abs + Real.1
                        lt_imp_pos_sub(q(i), Real.1)
                        Real.0 < Real.1 - q(i)
                        lt_imp_lte(Real.0, Real.1 - q(i))
                        Real.0 <= Real.1 - q(i)
                        mul_le_mul_of_nonneg_left(partial(abel_diff_abs(a, s), n1),
                            partial(abel_diff_abs(a, s), n1).abs + Real.1, Real.1 - q(i))
                        (Real.1 - q(i)) * partial(abel_diff_abs(a, s), n1) <= (Real.1 - q(i)) * (partial(abel_diff_abs(a, s), n1).abs + Real.1)
                        // (1 - q(i)) < delta, so (1 - q(i)) (c.abs + 1) < eps / 2.
                        abs_gte_zero(partial(abel_diff_abs(a, s), n1))
                        Real.0 <= partial(abel_diff_abs(a, s), n1).abs
                        lt_add_pos(partial(abel_diff_abs(a, s), n1).abs, Real.1)
                        partial(abel_diff_abs(a, s), n1).abs < partial(abel_diff_abs(a, s), n1).abs + Real.1
                        lte_lt_trans(Real.0, partial(abel_diff_abs(a, s), n1).abs,
                            partial(abel_diff_abs(a, s), n1).abs + Real.1)
                        Real.0 < partial(abel_diff_abs(a, s), n1).abs + Real.1
                        m <= n2
                        n2 <= i
                        lte_trans(m, n2, i)
                        m <= i
                        tail_bound_implies_is_close(q, Real.1, m,
                            (eps / two) / (partial(abel_diff_abs(a, s), n1).abs + Real.1), i)
                        q(i).is_close(Real.1, (eps / two) / (partial(abel_diff_abs(a, s), n1).abs + Real.1))
                        (q(i) - Real.1).abs < (eps / two) / (partial(abel_diff_abs(a, s), n1).abs + Real.1)
                        Real.1 - q(i) = Real.1 + -q(i)
                        Real.1 + -q(i) = -q(i) + Real.1
                        neg_distrib(q(i), -Real.1)
                        -(q(i) + -Real.1) = -q(i) + -(-Real.1)
                        neg_neg(Real.1)
                        -(-Real.1) = Real.1
                        -q(i) + -(-Real.1) = -q(i) + Real.1
                        -(q(i) - Real.1) = -(q(i) + -Real.1)
                        Real.1 - q(i) = -(q(i) - Real.1)
                        abs_neg(q(i) - Real.1)
                        (-(q(i) - Real.1)).abs = (q(i) - Real.1).abs
                        (Real.1 - q(i)).abs = (q(i) - Real.1).abs
                        lte_abs(Real.1 - q(i))
                        Real.1 - q(i) <= (Real.1 - q(i)).abs
                        Real.1 - q(i) <= (q(i) - Real.1).abs
                        (q(i) - Real.1).abs < (eps / two) / (partial(abel_diff_abs(a, s), n1).abs + Real.1)
                        lte_lt_trans(Real.1 - q(i), (q(i) - Real.1).abs,
                            (eps / two) / (partial(abel_diff_abs(a, s), n1).abs + Real.1))
                        Real.1 - q(i) < (eps / two) / (partial(abel_diff_abs(a, s), n1).abs + Real.1)
                        Real.0 < partial(abel_diff_abs(a, s), n1).abs + Real.1
                        mul_lt_mul_of_pos_right(Real.1 - q(i),
                            (eps / two) / (partial(abel_diff_abs(a, s), n1).abs + Real.1),
                            partial(abel_diff_abs(a, s), n1).abs + Real.1)
                        (Real.1 - q(i)) * (partial(abel_diff_abs(a, s), n1).abs + Real.1) < ((eps / two) / (partial(abel_diff_abs(a, s), n1).abs + Real.1)) *
                            (partial(abel_diff_abs(a, s), n1).abs + Real.1)
                        partial(abel_diff_abs(a, s), n1).abs + Real.1 != Real.0
                        mul_div_cancel(eps / two, partial(abel_diff_abs(a, s), n1).abs + Real.1)
                        (partial(abel_diff_abs(a, s), n1).abs + Real.1) *
                            ((eps / two) / (partial(abel_diff_abs(a, s), n1).abs + Real.1)) =
                            eps / two
                        real_mul_comm(partial(abel_diff_abs(a, s), n1).abs + Real.1,
                            (eps / two) / (partial(abel_diff_abs(a, s), n1).abs + Real.1))
                        ((eps / two) / (partial(abel_diff_abs(a, s), n1).abs + Real.1)) *
                            (partial(abel_diff_abs(a, s), n1).abs + Real.1) =
                            eps / two
                        (Real.1 - q(i)) * (partial(abel_diff_abs(a, s), n1).abs + Real.1) < eps / two
                        lte_lt_trans((Real.1 - q(i)) * partial(abel_diff_abs(a, s), n1),
                            (Real.1 - q(i)) * (partial(abel_diff_abs(a, s), n1).abs + Real.1),
                            eps / two)
                        (Real.1 - q(i)) * partial(abel_diff_abs(a, s), n1) < eps / two
                        // (eps / 2) q(i)^n1 <= eps / 2.
                        lt_imp_lte(q(i), Real.1)
                        q(i) <= Real.1
                        pow_le_one(q(i), n1)
                        q(i).pow(n1) <= Real.1
                        mul_le_mul_of_nonneg_left(q(i).pow(n1), Real.1, eps / two)
                        (eps / two) * q(i).pow(n1) <= (eps / two) * Real.1
                        (eps / two) * Real.1 = eps / two
                        (eps / two) * q(i).pow(n1) <= eps / two
                        // Sum the two bounds: < eps/2 + eps/2 = eps.
                        lt_add_right((Real.1 - q(i)) * partial(abel_diff_abs(a, s), n1),
                            eps / two, (eps / two) * q(i).pow(n1))
                        (Real.1 - q(i)) * partial(abel_diff_abs(a, s), n1) +
                            (eps / two) * q(i).pow(n1) < eps / two + (eps / two) * q(i).pow(n1)
                        lte_self(eps / two)
                        eps / two <= eps / two
                        add_lte_add(eps / two, eps / two, (eps / two) * q(i).pow(n1), eps / two)
                        eps / two + (eps / two) * q(i).pow(n1) <= eps / two + eps / two
                        lt_of_lt_of_lte((Real.1 - q(i)) * partial(abel_diff_abs(a, s), n1) +
                            (eps / two) * q(i).pow(n1),
                            eps / two + (eps / two) * q(i).pow(n1),
                            eps / two + eps / two)
                        (Real.1 - q(i)) * partial(abel_diff_abs(a, s), n1) +
                            (eps / two) * q(i).pow(n1) < eps / two + eps / two
                        lte_lt_trans((ps_sum(a, q(i)) - s).abs,
                            (Real.1 - q(i)) * partial(abel_diff_abs(a, s), n1) +
                            (eps / two) * q(i).pow(n1),
                            eps / two + eps / two)
                        (ps_sum(a, q(i)) - s).abs < eps / two + eps / two
                        half_plus_half(eps)
                        eps / two + eps / two = eps
                        (ps_sum(a, q(i)) - s).abs < eps
                        compose(ps_sum(a), q)(i) = ps_sum(a, q(i))
                        compose(ps_sum(a), q)(i).is_close(s, eps) =
                            (compose(ps_sum(a), q)(i) - s).abs < eps
                        (compose(ps_sum(a), q)(i) - s).abs = (ps_sum(a, q(i)) - s).abs
                        (compose(ps_sum(a), q)(i) - s).abs < eps
                        compose(ps_sum(a), q)(i).is_close(s, eps)
                    }
                }
                tail_bound(compose(ps_sum(a), q), s, n2, eps)
                exists(n: Nat) {
                    tail_bound(compose(ps_sum(a), q), s, n, eps)
                }
            }
        }
        // The loop above is exactly the definition of converges_to.
    }
}

/// Abel's theorem: if Σ a(n) converges, then Σ a(n) x^n tends to Σ a(n) as
/// x approaches 1 from below.
theorem abel_theorem(a: Nat -> Real, q: Nat -> Real) {
    converges(partial(a))
    and (forall(n: Nat) { Real.0 <= q(n) and q(n) < Real.1 })
    and converges_to(q, Real.1)
    implies converges_to(compose(ps_sum(a), q), limit(partial(a)))
} by {
    if converges(partial(a))
        and (forall(n: Nat) { Real.0 <= q(n) and q(n) < Real.1 })
        and converges_to(q, Real.1) {
        converges_imp_converges_to(partial(a))
        converges_to(partial(a), limit(partial(a)))
        abel_theorem_at(a, limit(partial(a)), q)
        converges_to(compose(ps_sum(a), q), limit(partial(a)))
    }
}
