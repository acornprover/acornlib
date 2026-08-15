/// Uniform convergence of sequences of real functions.
///
/// A sequence f_n of functions converges uniformly to g on an interval [a, b]
/// when every tolerance eps is eventually met at every point of the interval:
/// for every eps > 0 there is an index N such that |f_n(x) - g(x)| < eps for
/// every n >= N and every x in [a, b].  This file defines uniform convergence
/// (on an interval and on the whole line) together with pointwise convergence,
/// and proves the basic preservation theorems:
///   - uniform convergence implies pointwise convergence,
///   - the uniform limit of continuous functions is continuous (the eps/3
///     proof),
///   - the uniform limit is continuous at every interior point of the
///     interval.
/// Each defined predicate comes with an intro lemma (the pointwise form yields
/// the predicate) and an apply lemma (the predicate yields an instance), so
/// the definitions need not be unfolded at use sites.
///
/// The interchange of the uniform limit with the Riemann integral is proved at
/// the end of the file by an eps-sandwich argument: pointwise eps-closeness of
/// f_n to g on [a, b] keeps the lower and upper Darboux sums of f_n and g
/// within eps * (b - a) of each other over every partition, and the bounds
/// transfer to the integrals through the supremum/infimum of the sum sets.
/// The statement takes the integrability of g and of every f_n together with
/// explicit bounds on the sequence as hypotheses; deriving integrability of g
/// from the f_n alone would need the Darboux criterion with cross-partition
/// comparisons, which the Darboux foundation in real/integral.ac does not yet
/// provide.

from nat import Nat
from order import lt_imp_lte, lt_trans, lte_trans
from real.real_base import close_comm, close_imp_bounds, add_comm, lt_add_right,
    add_assoc, add_zero_left, add_neg_eq_zero, neg_neg, neg_distrib
from real.real_field import Real
numerals Real
numerals Nat
from real.real_seq import converges_to, tail_bound, is_close_triangle,
    close_and_lt_imp_close, find_less_than_a_third, eps_smaller_than_both,
    sub_lt_is_gt, lt_imp_minus_pos
from real.continuity_base import continuous, continuous_at, continuous_condition
from real.integral import interval_contains, interval_set, interval_image,
    interval_inf, interval_sup, interval_inf_spec, interval_sup_spec,
    lower_sum, upper_sum, lower_sum_set, upper_sum_set, lower_sum_contains,
    upper_sum_contains, partition_step_lower, partition_step_upper,
    is_partition, partition_start, partition_end, partition_mono,
    partition_point_in_interval, diff_step, telescope, partial_lte,
    image_lower_bound, interval_set_contains_left, interval_contains_left,
    interval_contains_right, interval_contains_mono, sub_nonneg, integral,
    is_integrable, integral_spec, set_supremum_le_upper_bound,
    set_member_le_supremum, set_supremum_is_upper_bound,
    set_upper_bound_contains_le, set_lower_bound_le_infimum,
    set_infimum_is_lower_bound, set_lower_bound_contains_le, has_lower_bound,
    has_upper_bound, is_nonempty, scalar_diff_step
from real.integral_exp import image_upper_bound, mul_frac_right
from real.double_sum import sub_seq, partial_sub_seq
from algebra.semigroup import mul_fn
from list import partial, partial_scalar_mul
from ordered_field import mul_le_mul_of_nonneg_right, mul_lt_mul_of_pos_right
from real.real_ring import mul_sub_distrib_left, mul_pos_pos
from real.am_gm import div_pos_of_pos_pos
from real.real_base import lte_lt_trans, bounds_imp_close,
    pos_gt_zero, gt_zero_imp_pos, lte_add_right, add_zero_right
from real.integrability import exists_n_frac_lt
from real.harmonic import from_nat_suc_pos_real, real_one_div_pos
from real.real_ring import mul_one_left
from real.supremum import function_image, function_image_contains,
    is_set_lower_bound, is_set_upper_bound, is_set_infimum, is_set_supremum
from data.basic.set import maps_into_set_image
from algebra.add_comm_group import sub_add_cancel
from algebra.add_ordered_group import add_le_add_right
from order import lt_imp_ne, lt_of_lt_of_lte, lt_of_lte_of_lt
from nat import lt_imp_lte_suc, from_nat

/// The n-th term of the function sequence f, evaluated at x.
define function_sequence_value(f: Nat -> Real -> Real, x: Real, n: Nat) -> Real {
    f(n, x)
}

/// True if the sequence of functions f converges uniformly to g on the
/// interval [a, b]: every tolerance is eventually met at every point of the
/// interval.
define uniform_converges_on(f: Nat -> Real -> Real, a: Real, b: Real, g: Real -> Real) -> Bool {
    forall(eps: Real) {
        eps.is_positive implies exists(n: Nat) {
            forall(i: Nat) {
                n <= i implies forall(x: Real) {
                    interval_contains(a, b, x) implies f(i, x).is_close(g(x), eps)
                }
            }
        }
    }
}

/// True if the sequence of functions f converges uniformly to g on the whole
/// real line.
define uniform_converges(f: Nat -> Real -> Real, g: Real -> Real) -> Bool {
    forall(eps: Real) {
        eps.is_positive implies exists(n: Nat) {
            forall(i: Nat) {
                n <= i implies forall(x: Real) {
                    f(i, x).is_close(g(x), eps)
                }
            }
        }
    }
}

/// True if the sequence of functions f converges pointwise to g at every point
/// of the interval [a, b].
define pointwise_converges_on(f: Nat -> Real -> Real, a: Real, b: Real, g: Real -> Real) -> Bool {
    forall(x: Real) {
        interval_contains(a, b, x) implies converges_to(function_sequence_value(f, x), g(x))
    }
}

/// True if the sequence of functions f converges pointwise to g everywhere.
define pointwise_converges(f: Nat -> Real -> Real, g: Real -> Real) -> Bool {
    forall(x: Real) {
        converges_to(function_sequence_value(f, x), g(x))
    }
}

// ---------------------------------------------------------------------------
// Intro and apply lemmas
// ---------------------------------------------------------------------------

/// The eps-form of uniform convergence on an interval implies the predicate.
theorem uniform_converges_on_intro(f: Nat -> Real -> Real, a: Real, b: Real, g: Real -> Real) {
    (forall(eps: Real) {
        eps.is_positive implies exists(n: Nat) {
            forall(i: Nat) {
                n <= i implies forall(x: Real) {
                    interval_contains(a, b, x) implies f(i, x).is_close(g(x), eps)
                }
            }
        }
    })
    implies uniform_converges_on(f, a, b, g)
} by {
    if forall(eps: Real) {
        eps.is_positive implies exists(n: Nat) {
            forall(i: Nat) {
                n <= i implies forall(x: Real) {
                    interval_contains(a, b, x) implies f(i, x).is_close(g(x), eps)
                }
            }
        }
    } {
        forall(eps: Real) {
            eps.is_positive implies exists(n: Nat) {
                forall(i: Nat) {
                    n <= i implies forall(x: Real) {
                        interval_contains(a, b, x) implies f(i, x).is_close(g(x), eps)
                    }
                }
            }
        }
        uniform_converges_on(f, a, b, g) = forall(eps: Real) {
            eps.is_positive implies exists(n: Nat) {
                forall(i: Nat) {
                    n <= i implies forall(x: Real) {
                        interval_contains(a, b, x) implies f(i, x).is_close(g(x), eps)
                    }
                }
            }
        }
    }
}

/// The predicate of uniform convergence on an interval applies at every
/// positive tolerance.
theorem uniform_converges_on_apply(f: Nat -> Real -> Real, a: Real, b: Real, g: Real -> Real, eps: Real) {
    uniform_converges_on(f, a, b, g) and eps.is_positive
    implies
    exists(n: Nat) {
        forall(i: Nat) {
            n <= i implies forall(x: Real) {
                interval_contains(a, b, x) implies f(i, x).is_close(g(x), eps)
            }
        }
    }
} by {
    if uniform_converges_on(f, a, b, g) and eps.is_positive {
        uniform_converges_on(f, a, b, g) = forall(eps2: Real) {
            eps2.is_positive implies exists(n2: Nat) {
                forall(i: Nat) {
                    n2 <= i implies forall(x: Real) {
                        interval_contains(a, b, x) implies f(i, x).is_close(g(x), eps2)
                    }
                }
            }
        }
        eps.is_positive implies exists(n: Nat) {
            forall(i: Nat) {
                n <= i implies forall(x: Real) {
                    interval_contains(a, b, x) implies f(i, x).is_close(g(x), eps)
                }
            }
        }
        let n: Nat satisfy {
            forall(i: Nat) {
                n <= i implies forall(x: Real) {
                    interval_contains(a, b, x) implies f(i, x).is_close(g(x), eps)
                }
            }
        }
        forall(i: Nat) {
            n <= i implies forall(x: Real) {
                interval_contains(a, b, x) implies f(i, x).is_close(g(x), eps)
            }
        }
    }
}

/// The eps-form of uniform convergence on the whole line implies the predicate.
theorem uniform_converges_intro(f: Nat -> Real -> Real, g: Real -> Real) {
    (forall(eps: Real) {
        eps.is_positive implies exists(n: Nat) {
            forall(i: Nat) {
                n <= i implies forall(x: Real) {
                    f(i, x).is_close(g(x), eps)
                }
            }
        }
    })
    implies uniform_converges(f, g)
} by {
    if forall(eps: Real) {
        eps.is_positive implies exists(n: Nat) {
            forall(i: Nat) {
                n <= i implies forall(x: Real) {
                    f(i, x).is_close(g(x), eps)
                }
            }
        }
    } {
        forall(eps: Real) {
            eps.is_positive implies exists(n: Nat) {
                forall(i: Nat) {
                    n <= i implies forall(x: Real) {
                        f(i, x).is_close(g(x), eps)
                    }
                }
            }
        }
        uniform_converges(f, g) = forall(eps: Real) {
            eps.is_positive implies exists(n: Nat) {
                forall(i: Nat) {
                    n <= i implies forall(x: Real) {
                        f(i, x).is_close(g(x), eps)
                    }
                }
            }
        }
    }
}

/// The predicate of uniform convergence on the whole line applies at every
/// positive tolerance.
theorem uniform_converges_apply(f: Nat -> Real -> Real, g: Real -> Real, eps: Real) {
    uniform_converges(f, g) and eps.is_positive
    implies
    exists(n: Nat) {
        forall(i: Nat) {
            n <= i implies forall(x: Real) {
                f(i, x).is_close(g(x), eps)
            }
        }
    }
} by {
    if uniform_converges(f, g) and eps.is_positive {
        uniform_converges(f, g) = forall(eps2: Real) {
            eps2.is_positive implies exists(n2: Nat) {
                forall(i: Nat) {
                    n2 <= i implies forall(x: Real) {
                        f(i, x).is_close(g(x), eps2)
                    }
                }
            }
        }
        eps.is_positive implies exists(n: Nat) {
            forall(i: Nat) {
                n <= i implies forall(x: Real) {
                    f(i, x).is_close(g(x), eps)
                }
            }
        }
        let n: Nat satisfy {
            forall(i: Nat) {
                n <= i implies forall(x: Real) {
                    f(i, x).is_close(g(x), eps)
                }
            }
        }
        forall(i: Nat) {
            n <= i implies forall(x: Real) {
                f(i, x).is_close(g(x), eps)
            }
        }
    }
}

/// The pointwise form of pointwise convergence on an interval implies the
/// predicate.
theorem pointwise_converges_on_intro(f: Nat -> Real -> Real, a: Real, b: Real, g: Real -> Real) {
    (forall(x: Real) {
        interval_contains(a, b, x) implies converges_to(function_sequence_value(f, x), g(x))
    })
    implies pointwise_converges_on(f, a, b, g)
} by {
    if forall(x: Real) {
        interval_contains(a, b, x) implies converges_to(function_sequence_value(f, x), g(x))
    } {
        forall(x: Real) {
            interval_contains(a, b, x) implies converges_to(function_sequence_value(f, x), g(x))
        }
        pointwise_converges_on(f, a, b, g) = forall(x: Real) {
            interval_contains(a, b, x) implies converges_to(function_sequence_value(f, x), g(x))
        }
    }
}

/// The predicate of pointwise convergence on an interval applies at each point
/// of the interval.
theorem pointwise_converges_on_apply(f: Nat -> Real -> Real, a: Real, b: Real, g: Real -> Real, x: Real) {
    pointwise_converges_on(f, a, b, g) and interval_contains(a, b, x)
    implies converges_to(function_sequence_value(f, x), g(x))
} by {
    if pointwise_converges_on(f, a, b, g) and interval_contains(a, b, x) {
        if not converges_to(function_sequence_value(f, x), g(x)) {
            not (interval_contains(a, b, x) implies converges_to(function_sequence_value(f, x), g(x)))
            not forall(y: Real) {
                interval_contains(a, b, y) implies converges_to(function_sequence_value(f, y), g(y))
            }
            pointwise_converges_on(f, a, b, g) = forall(y: Real) {
                interval_contains(a, b, y) implies converges_to(function_sequence_value(f, y), g(y))
            }
            not pointwise_converges_on(f, a, b, g)
            false
        }
        converges_to(function_sequence_value(f, x), g(x))
    }
}

/// The pointwise form of pointwise convergence on the whole line implies the
/// predicate.
theorem pointwise_converges_intro(f: Nat -> Real -> Real, g: Real -> Real) {
    (forall(x: Real) {
        converges_to(function_sequence_value(f, x), g(x))
    })
    implies pointwise_converges(f, g)
} by {
    if forall(x: Real) {
        converges_to(function_sequence_value(f, x), g(x))
    } {
        forall(x: Real) {
            converges_to(function_sequence_value(f, x), g(x))
        }
        pointwise_converges(f, g) = forall(x: Real) {
            converges_to(function_sequence_value(f, x), g(x))
        }
    }
}

/// The predicate of pointwise convergence on the whole line applies at each
/// point.
theorem pointwise_converges_apply(f: Nat -> Real -> Real, g: Real -> Real, x: Real) {
    pointwise_converges(f, g) implies converges_to(function_sequence_value(f, x), g(x))
} by {
    if pointwise_converges(f, g) {
        if not converges_to(function_sequence_value(f, x), g(x)) {
            not forall(y: Real) {
                converges_to(function_sequence_value(f, y), g(y))
            }
            pointwise_converges(f, g) = forall(y: Real) {
                converges_to(function_sequence_value(f, y), g(y))
            }
            not pointwise_converges(f, g)
            false
        }
        converges_to(function_sequence_value(f, x), g(x))
    }
}

// ---------------------------------------------------------------------------
// Uniform convergence implies pointwise convergence
// ---------------------------------------------------------------------------

/// Uniform convergence on an interval gives the tail bound of pointwise
/// convergence at each point of the interval, in unfolded form.
theorem uniform_converges_on_pointwise_tail(f: Nat -> Real -> Real, a: Real, b: Real, g: Real -> Real, x: Real) {
    uniform_converges_on(f, a, b, g) and interval_contains(a, b, x)
    implies
    forall(eps: Real) {
        eps.is_positive implies exists(n: Nat) {
            tail_bound(function_sequence_value(f, x), g(x), n, eps)
        }
    }
} by {
    if uniform_converges_on(f, a, b, g) and interval_contains(a, b, x) {
        uniform_converges_on(f, a, b, g) = forall(eps: Real) {
            eps.is_positive implies exists(n: Nat) {
                forall(i: Nat) {
                    n <= i implies forall(z: Real) {
                        interval_contains(a, b, z) implies f(i, z).is_close(g(z), eps)
                    }
                }
            }
        }
        forall(eps: Real) {
            if eps.is_positive {
                let n: Nat satisfy {
                    forall(i: Nat) {
                        n <= i implies forall(y: Real) {
                            interval_contains(a, b, y) implies f(i, y).is_close(g(y), eps)
                        }
                    }
                }
                forall(i: Nat) {
                    if n <= i {
                        n <= i implies forall(y: Real) {
                            interval_contains(a, b, y) implies f(i, y).is_close(g(y), eps)
                        }
                        forall(y: Real) {
                            interval_contains(a, b, y) implies f(i, y).is_close(g(y), eps)
                        }
                        interval_contains(a, b, x) implies f(i, x).is_close(g(x), eps)
                        f(i, x).is_close(g(x), eps)
                        function_sequence_value(f, x, i) = f(i, x)
                        function_sequence_value(f, x, i).is_close(g(x), eps)
                    }
                }
                tail_bound(function_sequence_value(f, x), g(x), n, eps) = forall(i: Nat) {
                    n <= i implies function_sequence_value(f, x, i).is_close(g(x), eps)
                }
                tail_bound(function_sequence_value(f, x), g(x), n, eps)
            }
        }
    }
}

/// Uniform convergence on an interval gives the pointwise convergence at each
/// point of the interval, in unfolded form.
theorem uniform_converges_on_pointwise_eps_form(f: Nat -> Real -> Real, a: Real, b: Real, g: Real -> Real) {
    uniform_converges_on(f, a, b, g)
    implies
    forall(x: Real) {
        interval_contains(a, b, x) implies converges_to(function_sequence_value(f, x), g(x))
    }
} by {
    if uniform_converges_on(f, a, b, g) {
        forall(x: Real) {
            if interval_contains(a, b, x) {
                uniform_converges_on_pointwise_tail(f, a, b, g, x)
                forall(eps: Real) {
                    eps.is_positive implies exists(n: Nat) {
                        tail_bound(function_sequence_value(f, x), g(x), n, eps)
                    }
                }
                converges_to(function_sequence_value(f, x), g(x))
            }
        }
    }
}

/// Uniform convergence on the interval implies pointwise convergence on the
/// interval.
theorem uniform_converges_on_imp_pointwise(f: Nat -> Real -> Real, a: Real, b: Real, g: Real -> Real) {
    uniform_converges_on(f, a, b, g) implies pointwise_converges_on(f, a, b, g)
} by {
    if uniform_converges_on(f, a, b, g) {
        uniform_converges_on_pointwise_eps_form(f, a, b, g)
        forall(x: Real) {
            interval_contains(a, b, x) implies converges_to(function_sequence_value(f, x), g(x))
        }
        pointwise_converges_on(f, a, b, g) = forall(x: Real) {
            interval_contains(a, b, x) implies converges_to(function_sequence_value(f, x), g(x))
        }
    }
}

/// Uniform convergence on the whole line gives the tail bound of pointwise
/// convergence at each point, in unfolded form.
theorem uniform_converges_pointwise_tail(f: Nat -> Real -> Real, g: Real -> Real, x: Real) {
    uniform_converges(f, g)
    implies
    forall(eps: Real) {
        eps.is_positive implies exists(n: Nat) {
            tail_bound(function_sequence_value(f, x), g(x), n, eps)
        }
    }
} by {
    if uniform_converges(f, g) {
        uniform_converges(f, g) = forall(eps: Real) {
            eps.is_positive implies exists(n: Nat) {
                forall(i: Nat) {
                    n <= i implies forall(z: Real) {
                        f(i, z).is_close(g(z), eps)
                    }
                }
            }
        }
        forall(eps: Real) {
            if eps.is_positive {
                let n: Nat satisfy {
                    forall(i: Nat) {
                        n <= i implies forall(y: Real) {
                            f(i, y).is_close(g(y), eps)
                        }
                    }
                }
                forall(i: Nat) {
                    if n <= i {
                        n <= i implies forall(y: Real) {
                            f(i, y).is_close(g(y), eps)
                        }
                        forall(y: Real) {
                            f(i, y).is_close(g(y), eps)
                        }
                        f(i, x).is_close(g(x), eps)
                        function_sequence_value(f, x, i) = f(i, x)
                        function_sequence_value(f, x, i).is_close(g(x), eps)
                    }
                }
                tail_bound(function_sequence_value(f, x), g(x), n, eps) = forall(i: Nat) {
                    n <= i implies function_sequence_value(f, x, i).is_close(g(x), eps)
                }
                tail_bound(function_sequence_value(f, x), g(x), n, eps)
            }
        }
    }
}

/// Uniform convergence on the whole line gives pointwise convergence at each
/// point, in unfolded form.
theorem uniform_converges_pointwise_eps_form(f: Nat -> Real -> Real, g: Real -> Real) {
    uniform_converges(f, g)
    implies
    forall(x: Real) {
        converges_to(function_sequence_value(f, x), g(x))
    }
} by {
    if uniform_converges(f, g) {
        forall(x: Real) {
            uniform_converges_pointwise_tail(f, g, x)
            forall(eps: Real) {
                eps.is_positive implies exists(n: Nat) {
                    tail_bound(function_sequence_value(f, x), g(x), n, eps)
                }
            }
            converges_to(function_sequence_value(f, x), g(x)) = forall(eps: Real) {
                eps.is_positive implies exists(n: Nat) {
                    tail_bound(function_sequence_value(f, x), g(x), n, eps)
                }
            }
        }
    }
}

/// Uniform convergence on the whole line implies pointwise convergence
/// everywhere.
theorem uniform_converges_imp_pointwise(f: Nat -> Real -> Real, g: Real -> Real) {
    uniform_converges(f, g) implies pointwise_converges(f, g)
} by {
    if uniform_converges(f, g) {
        uniform_converges_pointwise_eps_form(f, g)
        forall(x: Real) {
            converges_to(function_sequence_value(f, x), g(x))
        }
        pointwise_converges(f, g) = forall(x: Real) {
            converges_to(function_sequence_value(f, x), g(x))
        }
    }
}

// ---------------------------------------------------------------------------
// Restriction to an interval
// ---------------------------------------------------------------------------

/// Uniform convergence on the whole line restricts to an interval, in
/// unfolded form.
theorem uniform_converges_restricts_eps_form(f: Nat -> Real -> Real, a: Real, b: Real, g: Real -> Real) {
    uniform_converges(f, g)
    implies
    forall(eps: Real) {
        eps.is_positive implies exists(n: Nat) {
            forall(i: Nat) {
                n <= i implies forall(x: Real) {
                    interval_contains(a, b, x) implies f(i, x).is_close(g(x), eps)
                }
            }
        }
    }
} by {
    if uniform_converges(f, g) {
        uniform_converges(f, g) = forall(eps: Real) {
            eps.is_positive implies exists(n: Nat) {
                forall(i: Nat) {
                    n <= i implies forall(x: Real) {
                        f(i, x).is_close(g(x), eps)
                    }
                }
            }
        }
        forall(eps: Real) {
            if eps.is_positive {
                let n: Nat satisfy {
                    forall(i: Nat) {
                        n <= i implies forall(x: Real) {
                            f(i, x).is_close(g(x), eps)
                        }
                    }
                }
                forall(i: Nat) {
                    n <= i implies forall(x: Real) {
                        interval_contains(a, b, x) implies f(i, x).is_close(g(x), eps)
                    }
                }
            }
        }
    }
}

/// Uniform convergence on the whole line restricts to uniform convergence on
/// any interval.
theorem uniform_converges_imp_uniform_converges_on(f: Nat -> Real -> Real, a: Real, b: Real, g: Real -> Real) {
    uniform_converges(f, g) implies uniform_converges_on(f, a, b, g)
} by {
    if uniform_converges(f, g) {
        uniform_converges_restricts_eps_form(f, a, b, g)
        forall(eps: Real) {
            eps.is_positive implies exists(n: Nat) {
                forall(i: Nat) {
                    n <= i implies forall(x: Real) {
                        interval_contains(a, b, x) implies f(i, x).is_close(g(x), eps)
                    }
                }
            }
        }
        uniform_converges_on(f, a, b, g) = forall(eps: Real) {
            eps.is_positive implies exists(n: Nat) {
                forall(i: Nat) {
                    n <= i implies forall(x: Real) {
                        interval_contains(a, b, x) implies f(i, x).is_close(g(x), eps)
                    }
                }
            }
        }
    }
}

/// Pointwise convergence on the whole line restricts to pointwise convergence
/// on any interval.
theorem pointwise_converges_imp_pointwise_converges_on(f: Nat -> Real -> Real, a: Real, b: Real, g: Real -> Real) {
    pointwise_converges(f, g) implies pointwise_converges_on(f, a, b, g)
} by {
    if pointwise_converges(f, g) {
        forall(x: Real) {
            if interval_contains(a, b, x) {
                pointwise_converges_apply(f, g, x)
                converges_to(function_sequence_value(f, x), g(x))
            }
        }
        pointwise_converges_on_intro(f, a, b, g)
        pointwise_converges_on(f, a, b, g)
    }
}

// ---------------------------------------------------------------------------
// The uniform limit of continuous functions is continuous
// ---------------------------------------------------------------------------

/// The uniform limit of a sequence of continuous functions is continuous, in
/// unfolded form: at every point and every tolerance there is a delta.
theorem uniform_limit_continuous_eps_form(f: Nat -> Real -> Real, g: Real -> Real) {
    uniform_converges(f, g) and (forall(n: Nat) { continuous(f(n)) })
    implies
    forall(x0: Real) {
        forall(eps: Real) {
            eps.is_positive implies exists(delta: Real) {
                delta.is_positive and continuous_condition(g, x0, delta, eps)
            }
        }
    }
} by {
    if uniform_converges(f, g) and (forall(n: Nat) { continuous(f(n)) }) {
        uniform_converges(f, g) = forall(eps: Real) {
            eps.is_positive implies exists(n: Nat) {
                forall(i: Nat) {
                    n <= i implies forall(x: Real) {
                        f(i, x).is_close(g(x), eps)
                    }
                }
            }
        }
        forall(x0: Real) {
            forall(eps: Real) {
                if eps.is_positive {
                    let eps3: Real satisfy {
                        eps3.is_positive and eps3 + eps3 + eps3 < eps
                    }
                    let n: Nat satisfy {
                        forall(i: Nat) {
                            n <= i implies forall(x: Real) {
                                f(i, x).is_close(g(x), eps3)
                            }
                        }
                    }
                    n <= n
                    n <= n implies forall(x: Real) {
                        f(n, x).is_close(g(x), eps3)
                    }
                    forall(x: Real) {
                        f(n, x).is_close(g(x), eps3)
                    }
                    f(n, x0).is_close(g(x0), eps3)
                    continuous(f(n))
                    continuous(f(n)) = forall(y: Real) {
                        continuous_at(f(n), y)
                    }
                    continuous_at(f(n), x0)
                    continuous_at(f(n), x0) = forall(eps2: Real) {
                        eps2.is_positive implies exists(delta: Real) {
                            delta.is_positive and continuous_condition(f(n), x0, delta, eps2)
                        }
                    }
                    let delta: Real satisfy {
                        delta.is_positive and continuous_condition(f(n), x0, delta, eps3)
                    }
                    continuous_condition(f(n), x0, delta, eps3) = forall(x1: Real) {
                        x1.is_close(x0, delta) implies f(n, x1).is_close(f(n, x0), eps3)
                    }
                    forall(x: Real) {
                        if x.is_close(x0, delta) {
                            x.is_close(x0, delta) implies f(n, x).is_close(f(n, x0), eps3)
                            f(n, x).is_close(f(n, x0), eps3)
                            forall(y: Real) {
                                f(n, y).is_close(g(y), eps3)
                            }
                            f(n, x).is_close(g(x), eps3)
                            close_comm(f(n, x), f(n, x0), eps3)
                            f(n, x0).is_close(f(n, x), eps3)
                            is_close_triangle(g(x), f(n, x0), f(n, x), eps3, eps3)
                            g(x).is_close(f(n, x0), eps3 + eps3)
                            close_comm(g(x0), f(n, x0), eps3)
                            g(x0).is_close(f(n, x0), eps3)
                            is_close_triangle(g(x), g(x0), f(n, x0), eps3 + eps3, eps3)
                            g(x).is_close(g(x0), eps3 + eps3 + eps3)
                            eps3 + eps3 + eps3 < eps
                            close_and_lt_imp_close(g(x), g(x0), eps3 + eps3 + eps3, eps)
                            g(x).is_close(g(x0), eps)
                        }
                    }
                    continuous_condition(g, x0, delta, eps) = forall(x1: Real) {
                        x1.is_close(x0, delta) implies g(x1).is_close(g(x0), eps)
                    }
                    continuous_condition(g, x0, delta, eps)
                    delta.is_positive and continuous_condition(g, x0, delta, eps)
                }
            }
        }
    }
}

/// The uniform limit of continuous functions is continuous.
theorem uniform_limit_continuous(f: Nat -> Real -> Real, g: Real -> Real) {
    uniform_converges(f, g) and (forall(n: Nat) { continuous(f(n)) })
    implies
    continuous(g)
} by {
    if uniform_converges(f, g) and (forall(n: Nat) { continuous(f(n)) }) {
        uniform_limit_continuous_eps_form(f, g)
        forall(x0: Real) {
            forall(eps: Real) {
                eps.is_positive implies exists(delta: Real) {
                    delta.is_positive and continuous_condition(g, x0, delta, eps)
                }
            }
        }
        continuous(g) = forall(x0: Real) {
            forall(eps: Real) {
                eps.is_positive implies exists(delta: Real) {
                    delta.is_positive and continuous_condition(g, x0, delta, eps)
                }
            }
        }
    }
}

// ---------------------------------------------------------------------------
// Continuity of the uniform limit on the interior of the interval
// ---------------------------------------------------------------------------

/// The uniform limit of continuous functions is continuous at every interior
/// point of the interval, in unfolded form.
theorem uniform_limit_continuous_on_eps_form(f: Nat -> Real -> Real, a: Real, b: Real, g: Real -> Real) {
    uniform_converges_on(f, a, b, g) and (forall(n: Nat) { continuous(f(n)) })
    implies
    forall(x0: Real) {
        (a < x0 and x0 < b) implies
        forall(eps: Real) {
            eps.is_positive implies exists(delta: Real) {
                delta.is_positive and continuous_condition(g, x0, delta, eps)
            }
        }
    }
} by {
    if uniform_converges_on(f, a, b, g) and (forall(n: Nat) { continuous(f(n)) }) {
        uniform_converges_on(f, a, b, g) = forall(eps: Real) {
            eps.is_positive implies exists(n: Nat) {
                forall(i: Nat) {
                    n <= i implies forall(x: Real) {
                        interval_contains(a, b, x) implies f(i, x).is_close(g(x), eps)
                    }
                }
            }
        }
        forall(x0: Real) {
            if a < x0 and x0 < b {
                forall(eps: Real) {
                    if eps.is_positive {
                        let eps3: Real satisfy {
                            eps3.is_positive and eps3 + eps3 + eps3 < eps
                        }
                        let n: Nat satisfy {
                            forall(i: Nat) {
                                n <= i implies forall(x: Real) {
                                    interval_contains(a, b, x) implies f(i, x).is_close(g(x), eps3)
                                }
                            }
                        }
                        n <= n
                        lt_imp_lte(a, x0)
                        a <= x0
                        lt_imp_lte(x0, b)
                        x0 <= b
                        a <= x0 and x0 <= b
                        interval_contains(a, b, x0) = a <= x0 and x0 <= b
                        interval_contains(a, b, x0)
                        n <= n implies forall(x: Real) {
                            interval_contains(a, b, x) implies f(n, x).is_close(g(x), eps3)
                        }
                        forall(x: Real) {
                            interval_contains(a, b, x) implies f(n, x).is_close(g(x), eps3)
                        }
                        f(n, x0).is_close(g(x0), eps3)
                        continuous(f(n))
                        continuous(f(n)) = forall(y: Real) {
                            continuous_at(f(n), y)
                        }
                        continuous_at(f(n), x0)
                        continuous_at(f(n), x0) = forall(eps2: Real) {
                            eps2.is_positive implies exists(delta: Real) {
                                delta.is_positive and continuous_condition(f(n), x0, delta, eps2)
                            }
                        }
                        let delta_cont: Real satisfy {
                            delta_cont.is_positive and continuous_condition(f(n), x0, delta_cont, eps3)
                        }
                        lt_imp_minus_pos(x0, b)
                        (b - x0).is_positive
                        lt_imp_minus_pos(a, x0)
                        (x0 - a).is_positive
                        eps_smaller_than_both(delta_cont, b - x0)
                        let d1: Real satisfy {
                            d1.is_positive and d1 < delta_cont and d1 < b - x0
                        }
                        eps_smaller_than_both(d1, x0 - a)
                        let delta: Real satisfy {
                            delta.is_positive and delta < d1 and delta < x0 - a
                        }
                        lt_trans(delta, d1, delta_cont)
                        delta < delta_cont
                        lt_trans(delta, d1, b - x0)
                        delta < b - x0
                        delta < x0 - a
                        continuous_condition(f(n), x0, delta_cont, eps3) = forall(x1: Real) {
                            x1.is_close(x0, delta_cont) implies f(n, x1).is_close(f(n, x0), eps3)
                        }
                        forall(x: Real) {
                            if x.is_close(x0, delta) {
                                close_imp_bounds(x, x0, delta)
                                x > x0 - delta
                                sub_lt_is_gt(x0, delta, x0 - a)
                                x0 - delta > x0 - (x0 - a)
                                x0 - (x0 - a) = x0 + -(x0 - a)
                                x0 - a = x0 + -a
                                -(x0 - a) = -(x0 + -a)
                                neg_distrib(x0, -a)
                                -(x0 + -a) = -x0 + -(-a)
                                neg_neg(a)
                                -(-a) = a
                                -x0 + -(-a) = -x0 + a
                                -(x0 - a) = -x0 + a
                                x0 - (x0 - a) = x0 + (-x0 + a)
                                add_assoc(x0, -x0, a)
                                x0 + (-x0 + a) = (x0 + -x0) + a
                                x0 - (x0 - a) = (x0 + -x0) + a
                                add_neg_eq_zero(x0)
                                x0 + -x0 = Real.0
                                (x0 + -x0) + a = Real.0 + a
                                add_zero_left(a)
                                Real.0 + a = a
                                x0 - (x0 - a) = a
                                x0 - delta > a
                                lt_trans(a, x0 - delta, x)
                                a < x
                                lt_imp_lte(a, x)
                                a <= x
                                x < x0 + delta
                                lt_add_right(delta, b - x0, x0)
                                delta + x0 < (b - x0) + x0
                                add_comm(delta, x0)
                                delta + x0 = x0 + delta
                                (b - x0) + x0 = b
                                x0 + delta < b
                                lt_imp_lte(x, x0 + delta)
                                x <= x0 + delta
                                lt_imp_lte(x0 + delta, b)
                                x0 + delta <= b
                                lte_trans(x, x0 + delta, b)
                                x <= b
                                a <= x and x <= b
                                interval_contains(a, b, x) = a <= x and x <= b
                                interval_contains(a, b, x)
                                close_and_lt_imp_close(x, x0, delta, delta_cont)
                                x.is_close(x0, delta_cont)
                                x.is_close(x0, delta_cont) implies f(n, x).is_close(f(n, x0), eps3)
                                f(n, x).is_close(f(n, x0), eps3)
                                forall(y: Real) {
                                    interval_contains(a, b, y) implies f(n, y).is_close(g(y), eps3)
                                }
                                interval_contains(a, b, x) implies f(n, x).is_close(g(x), eps3)
                                f(n, x).is_close(g(x), eps3)
                                close_comm(f(n, x), f(n, x0), eps3)
                                f(n, x0).is_close(f(n, x), eps3)
                                is_close_triangle(g(x), f(n, x0), f(n, x), eps3, eps3)
                                g(x).is_close(f(n, x0), eps3 + eps3)
                                close_comm(g(x0), f(n, x0), eps3)
                                g(x0).is_close(f(n, x0), eps3)
                                is_close_triangle(g(x), g(x0), f(n, x0), eps3 + eps3, eps3)
                                g(x).is_close(g(x0), eps3 + eps3 + eps3)
                                eps3 + eps3 + eps3 < eps
                                close_and_lt_imp_close(g(x), g(x0), eps3 + eps3 + eps3, eps)
                                g(x).is_close(g(x0), eps)
                            }
                        }
                        continuous_condition(g, x0, delta, eps) = forall(x1: Real) {
                            x1.is_close(x0, delta) implies g(x1).is_close(g(x0), eps)
                        }
                        continuous_condition(g, x0, delta, eps)
                        delta.is_positive and continuous_condition(g, x0, delta, eps)
                    }
                }
            }
        }
    }
}

/// The uniform limit of continuous functions is continuous at every interior
/// point of the interval.
theorem uniform_limit_continuous_on(f: Nat -> Real -> Real, a: Real, b: Real, g: Real -> Real) {
    uniform_converges_on(f, a, b, g) and (forall(n: Nat) { continuous(f(n)) })
    implies
    forall(x0: Real) {
        (a < x0 and x0 < b) implies continuous_at(g, x0)
    }
} by {
    if uniform_converges_on(f, a, b, g) and (forall(n: Nat) { continuous(f(n)) }) {
        uniform_limit_continuous_on_eps_form(f, a, b, g)
        forall(x0: Real) {
            if a < x0 and x0 < b {
                forall(eps: Real) {
                    eps.is_positive implies exists(delta: Real) {
                        delta.is_positive and continuous_condition(g, x0, delta, eps)
                    }
                }
                continuous_at(g, x0)
            }
        }
    }
}

// ---------------------------------------------------------------------------
// Darboux-sum eps-sandwich lemmas
// ---------------------------------------------------------------------------
//
// The helper lemmas behind the interchange theorem: eps-scaling
// (exists_eps_mul_lt), one-sided rearrangements of inequalities, and the
// per-cell / per-partition bounds that pointwise eps-closeness imposes on the
// Darboux sums (interval_inf_eps_close, interval_sup_eps_close,
// lower_sum_eps_close, upper_sum_eps_close).

/// The sequence of integrals of the terms of a function sequence.
define integral_sequence(f: Nat -> Real -> Real, a: Real, b: Real, n: Nat) -> Real {
    integral(f(n), a, b)
}

/// If a - b <= c, then a <= c + b.
theorem lte_sub_imp_lte_add(a: Real, b: Real, c: Real) {
    a - b <= c implies a <= c + b
} by {
    if a - b <= c {
        (a - b) + b <= c + b
        (a - b) + b = a
        a <= c + b
    }
}

/// If a <= b + c, then a - c <= b.
theorem lte_imp_lte_sub(a: Real, b: Real, c: Real) {
    a <= b + c implies a - c <= b
} by {
    if a <= b + c {
        add_le_add_right[Real](a, b + c, -c)
        a + -c <= b + c + -c
        a + -c = a - c
        add_assoc(b, c, -c)
        b + c + -c = b + (c + -c)
        add_neg_eq_zero(c)
        c + -c = Real.0
        b + (c + -c) = b + Real.0
        add_zero_right(b)
        b + Real.0 = b
        a - c <= b
    }
}

/// For a nonnegative length m and a positive tolerance eps, some positive
/// eps3 has eps3 * m < eps.
theorem exists_eps_mul_lt(m: Real, eps: Real) {
    Real.0 <= m and eps.is_positive
    implies
    exists(eps3: Real) {
        eps3.is_positive and eps3 * m < eps
    }
} by {
    if Real.0 <= m and eps.is_positive {
        exists_n_frac_lt(m, eps)
        exists(n: Nat) {
            m / from_nat[Real](n.suc) < eps
        }
        let n: Nat satisfy {
            m / from_nat[Real](n.suc) < eps
        }
        from_nat_suc_pos_real(n)
        from_nat[Real](n.suc) > Real.0
        real_one_div_pos(from_nat[Real](n.suc))
        Real.1 / from_nat[Real](n.suc) > Real.0
        gt_zero_imp_pos(Real.1 / from_nat[Real](n.suc))
        (Real.1 / from_nat[Real](n.suc)).is_positive
        Real.0 < from_nat[Real](n.suc)
        lt_imp_ne(Real.0, from_nat[Real](n.suc))
        Real.0 != from_nat[Real](n.suc)
        from_nat[Real](n.suc) != Real.0
        mul_frac_right(Real.1, from_nat[Real](n.suc), m)
        (Real.1 / from_nat[Real](n.suc)) * m = (Real.1 * m) / from_nat[Real](n.suc)
        mul_one_left(m)
        Real.1 * m = m
        (Real.1 / from_nat[Real](n.suc)) * m = m / from_nat[Real](n.suc)
        m / from_nat[Real](n.suc) < eps
        (Real.1 / from_nat[Real](n.suc)) * m < eps
        (Real.1 / from_nat[Real](n.suc)).is_positive and (Real.1 / from_nat[Real](n.suc)) * m < eps
        exists(eps3: Real) {
            eps3.is_positive and eps3 * m < eps
        }
    }
}

// ---------------------------------------------------------------------------
// Darboux-sum eps-sandwich lemmas
// ---------------------------------------------------------------------------
//
// If g - eps <= f pointwise on a cell, the infimum of f over the cell stays
// within eps of the infimum of g; dually, if f <= g + eps pointwise, the
// supremum of f stays within eps of the supremum of g.  Summed over the cells
// of a partition these give the lower- and upper-sum sandwich used by the
// interchange theorem.

/// If g - eps <= f pointwise on [x, y], the infimum of f is within eps above
/// the infimum of g.
theorem interval_inf_eps_close(f: Real -> Real, g: Real -> Real, x: Real, y: Real, eps: Real, lb_f: Real, lb_g: Real) {
    x <= y and
    (forall(t: Real) { interval_contains(x, y, t) implies lb_f <= f(t) }) and
    (forall(t: Real) { interval_contains(x, y, t) implies lb_g <= g(t) }) and
    (forall(t: Real) { interval_contains(x, y, t) implies g(t) - eps <= f(t) })
    implies
    interval_inf(g, x, y) - eps <= interval_inf(f, x, y)
} by {
    if x <= y and
       (forall(t: Real) { interval_contains(x, y, t) implies lb_f <= f(t) }) and
       (forall(t: Real) { interval_contains(x, y, t) implies lb_g <= g(t) }) and
       (forall(t: Real) { interval_contains(x, y, t) implies g(t) - eps <= f(t) }) {
        interval_set_contains_left(x, y)
        is_nonempty(interval_set(x, y))
        image_lower_bound(f, x, y, lb_f)
        is_set_lower_bound(interval_image(f, x, y), lb_f)
        exists(b: Real) {
            is_set_lower_bound(interval_image(f, x, y), b)
        }
        has_lower_bound(interval_image(f, x, y))
        is_nonempty(interval_set(x, y)) and has_lower_bound(interval_image(f, x, y))
        interval_inf_spec(f, x, y)
        is_set_infimum(interval_image(f, x, y), interval_inf(f, x, y))
        image_lower_bound(g, x, y, lb_g)
        is_set_lower_bound(interval_image(g, x, y), lb_g)
        exists(b0: Real) {
            is_set_lower_bound(interval_image(g, x, y), b0)
        }
        has_lower_bound(interval_image(g, x, y))
        is_nonempty(interval_set(x, y)) and has_lower_bound(interval_image(g, x, y))
        interval_inf_spec(g, x, y)
        is_set_infimum(interval_image(g, x, y), interval_inf(g, x, y))
        set_infimum_is_lower_bound(interval_image(g, x, y), interval_inf(g, x, y))
        is_set_lower_bound(interval_image(g, x, y), interval_inf(g, x, y))
        forall(v: Real) {
            if interval_image(f, x, y).contains(v) {
                interval_image(f, x, y).contains(v) = function_image_contains(f, interval_set(x, y), v)
                function_image_contains(f, interval_set(x, y), v)
                let t: Real satisfy {
                    interval_set(x, y).contains(t) and v = f(t)
                }
                interval_set(x, y).contains(t) = interval_contains(x, y, t)
                interval_contains(x, y, t)
                forall(t0: Real) {
                    interval_contains(x, y, t0) implies g(t0) - eps <= f(t0)
                }
                interval_contains(x, y, t) implies g(t) - eps <= f(t)
                g(t) - eps <= f(t)
                maps_into_set_image(interval_set(x, y), g, t)
                function_image(g, interval_set(x, y)).contains(g(t))
                interval_image(g, x, y).contains(g(t))
                set_lower_bound_contains_le(interval_image(g, x, y), interval_inf(g, x, y), g(t))
                interval_inf(g, x, y) <= g(t)
                add_le_add_right[Real](interval_inf(g, x, y), g(t), -eps)
                interval_inf(g, x, y) + -eps <= g(t) + -eps
                interval_inf(g, x, y) + -eps = interval_inf(g, x, y) - eps
                g(t) + -eps = g(t) - eps
                interval_inf(g, x, y) - eps <= g(t) - eps
                lte_trans[Real](interval_inf(g, x, y) - eps, g(t) - eps, f(t))
                interval_inf(g, x, y) - eps <= f(t)
                v = f(t)
                interval_inf(g, x, y) - eps <= v
            }
        }
        is_set_lower_bound(interval_image(f, x, y), interval_inf(g, x, y) - eps)
        set_lower_bound_le_infimum(interval_image(f, x, y), interval_inf(f, x, y), interval_inf(g, x, y) - eps)
        interval_inf(g, x, y) - eps <= interval_inf(f, x, y)
    }
}

/// If f <= g + eps pointwise on [x, y], the supremum of f is within eps above
/// the supremum of g.
theorem interval_sup_eps_close(f: Real -> Real, g: Real -> Real, x: Real, y: Real, eps: Real, ub_f: Real, ub_g: Real) {
    x <= y and
    (forall(t: Real) { interval_contains(x, y, t) implies f(t) <= ub_f }) and
    (forall(t: Real) { interval_contains(x, y, t) implies g(t) <= ub_g }) and
    (forall(t: Real) { interval_contains(x, y, t) implies f(t) <= g(t) + eps })
    implies
    interval_sup(f, x, y) - eps <= interval_sup(g, x, y)
} by {
    if x <= y and
       (forall(t: Real) { interval_contains(x, y, t) implies f(t) <= ub_f }) and
       (forall(t: Real) { interval_contains(x, y, t) implies g(t) <= ub_g }) and
       (forall(t: Real) { interval_contains(x, y, t) implies f(t) <= g(t) + eps }) {
        interval_set_contains_left(x, y)
        is_nonempty(interval_set(x, y))
        image_upper_bound(f, x, y, ub_f)
        is_set_upper_bound(interval_image(f, x, y), ub_f)
        exists(b: Real) {
            is_set_upper_bound(interval_image(f, x, y), b)
        }
        has_upper_bound(interval_image(f, x, y))
        is_nonempty(interval_set(x, y)) and has_upper_bound(interval_image(f, x, y))
        interval_sup_spec(f, x, y)
        is_set_supremum(interval_image(f, x, y), interval_sup(f, x, y))
        image_upper_bound(g, x, y, ub_g)
        is_set_upper_bound(interval_image(g, x, y), ub_g)
        exists(b0: Real) {
            is_set_upper_bound(interval_image(g, x, y), b0)
        }
        has_upper_bound(interval_image(g, x, y))
        is_nonempty(interval_set(x, y)) and has_upper_bound(interval_image(g, x, y))
        interval_sup_spec(g, x, y)
        is_set_supremum(interval_image(g, x, y), interval_sup(g, x, y))
        set_supremum_is_upper_bound(interval_image(g, x, y), interval_sup(g, x, y))
        is_set_upper_bound(interval_image(g, x, y), interval_sup(g, x, y))
        forall(v: Real) {
            if interval_image(f, x, y).contains(v) {
                interval_image(f, x, y).contains(v) = function_image_contains(f, interval_set(x, y), v)
                function_image_contains(f, interval_set(x, y), v)
                let t: Real satisfy {
                    interval_set(x, y).contains(t) and v = f(t)
                }
                interval_set(x, y).contains(t) = interval_contains(x, y, t)
                interval_contains(x, y, t)
                forall(t0: Real) {
                    interval_contains(x, y, t0) implies f(t0) <= g(t0) + eps
                }
                interval_contains(x, y, t) implies f(t) <= g(t) + eps
                f(t) <= g(t) + eps
                maps_into_set_image(interval_set(x, y), g, t)
                function_image(g, interval_set(x, y)).contains(g(t))
                interval_image(g, x, y).contains(g(t))
                set_upper_bound_contains_le(interval_image(g, x, y), interval_sup(g, x, y), g(t))
                g(t) <= interval_sup(g, x, y)
                add_le_add_right[Real](g(t), interval_sup(g, x, y), eps)
                g(t) + eps <= interval_sup(g, x, y) + eps
                lte_trans[Real](f(t), g(t) + eps, interval_sup(g, x, y) + eps)
                f(t) <= interval_sup(g, x, y) + eps
                v = f(t)
                v <= interval_sup(g, x, y) + eps
            }
        }
        is_set_upper_bound(interval_image(f, x, y), interval_sup(g, x, y) + eps)
        set_supremum_le_upper_bound(interval_image(f, x, y), interval_sup(f, x, y), interval_sup(g, x, y) + eps)
        interval_sup(f, x, y) <= interval_sup(g, x, y) + eps
        lte_imp_lte_sub(interval_sup(f, x, y), interval_sup(g, x, y), eps)
        interval_sup(f, x, y) - eps <= interval_sup(g, x, y)
    }
}

/// If g - eps <= f pointwise on [a, b], every lower sum of g is within
/// eps * (b - a) of the corresponding lower sum of f.
theorem lower_sum_eps_close(f: Real -> Real, g: Real -> Real, p: Nat -> Real, a: Real, b: Real, n: Nat, eps: Real, lb_f: Real, lb_g: Real) {
    is_partition(p, a, b, n) and
    (forall(t: Real) { interval_contains(a, b, t) implies lb_f <= f(t) }) and
    (forall(t: Real) { interval_contains(a, b, t) implies lb_g <= g(t) }) and
    (forall(t: Real) { interval_contains(a, b, t) implies g(t) - eps <= f(t) })
    implies
    lower_sum(g, p, n) <= lower_sum(f, p, n) + eps * (b - a)
} by {
    if is_partition(p, a, b, n) and
       (forall(t: Real) { interval_contains(a, b, t) implies lb_f <= f(t) }) and
       (forall(t: Real) { interval_contains(a, b, t) implies lb_g <= g(t) }) and
       (forall(t: Real) { interval_contains(a, b, t) implies g(t) - eps <= f(t) }) {
        forall(k: Nat) {
            if k < n {
                lt_imp_lte_suc(k, n)
                k + 1 <= n
                k <= k + 1
                partition_mono(p, a, b, n, k, k + 1)
                p(k) <= p(k + 1)
                partition_point_in_interval(p, a, b, n, k)
                interval_contains(a, b, p(k))
                partition_point_in_interval(p, a, b, n, k + 1)
                interval_contains(a, b, p(k + 1))
                forall(t: Real) {
                    if interval_contains(p(k), p(k + 1), t) {
                        interval_contains_left(p(k), p(k + 1), t)
                        p(k) <= t
                        interval_contains_right(p(k), p(k + 1), t)
                        t <= p(k + 1)
                        interval_contains_mono(a, b, p(k), p(k + 1), t)
                        interval_contains(a, b, t)
                        forall(t0: Real) {
                            interval_contains(a, b, t0) implies lb_f <= f(t0)
                        }
                        interval_contains(a, b, t) implies lb_f <= f(t)
                        lb_f <= f(t)
                    }
                }
                forall(t: Real) {
                    if interval_contains(p(k), p(k + 1), t) {
                        interval_contains_left(p(k), p(k + 1), t)
                        p(k) <= t
                        interval_contains_right(p(k), p(k + 1), t)
                        t <= p(k + 1)
                        interval_contains_mono(a, b, p(k), p(k + 1), t)
                        interval_contains(a, b, t)
                        forall(t1: Real) {
                            interval_contains(a, b, t1) implies lb_g <= g(t1)
                        }
                        interval_contains(a, b, t) implies lb_g <= g(t)
                        lb_g <= g(t)
                    }
                }
                forall(t: Real) {
                    if interval_contains(p(k), p(k + 1), t) {
                        interval_contains_left(p(k), p(k + 1), t)
                        p(k) <= t
                        interval_contains_right(p(k), p(k + 1), t)
                        t <= p(k + 1)
                        interval_contains_mono(a, b, p(k), p(k + 1), t)
                        interval_contains(a, b, t)
                        forall(t2: Real) {
                            interval_contains(a, b, t2) implies g(t2) - eps <= f(t2)
                        }
                        interval_contains(a, b, t) implies g(t) - eps <= f(t)
                        g(t) - eps <= f(t)
                    }
                }
                forall(t: Real) {
                    interval_contains(p(k), p(k + 1), t) implies lb_f <= f(t)
                }
                forall(t: Real) {
                    interval_contains(p(k), p(k + 1), t) implies lb_g <= g(t)
                }
                forall(t: Real) {
                    interval_contains(p(k), p(k + 1), t) implies g(t) - eps <= f(t)
                }
                interval_set_contains_left(p(k), p(k + 1))
                is_nonempty(interval_set(p(k), p(k + 1)))
                image_lower_bound(f, p(k), p(k + 1), lb_f)
                is_set_lower_bound(interval_image(f, p(k), p(k + 1)), lb_f)
                exists(b1: Real) {
                    is_set_lower_bound(interval_image(f, p(k), p(k + 1)), b1)
                }
                has_lower_bound(interval_image(f, p(k), p(k + 1)))
                is_nonempty(interval_set(p(k), p(k + 1))) and has_lower_bound(interval_image(f, p(k), p(k + 1)))
                interval_inf_spec(f, p(k), p(k + 1))
                is_set_infimum(interval_image(f, p(k), p(k + 1)), interval_inf(f, p(k), p(k + 1)))
                image_lower_bound(g, p(k), p(k + 1), lb_g)
                is_set_lower_bound(interval_image(g, p(k), p(k + 1)), lb_g)
                exists(b2: Real) {
                    is_set_lower_bound(interval_image(g, p(k), p(k + 1)), b2)
                }
                has_lower_bound(interval_image(g, p(k), p(k + 1)))
                is_nonempty(interval_set(p(k), p(k + 1))) and has_lower_bound(interval_image(g, p(k), p(k + 1)))
                interval_inf_spec(g, p(k), p(k + 1))
                is_set_infimum(interval_image(g, p(k), p(k + 1)), interval_inf(g, p(k), p(k + 1)))
                set_infimum_is_lower_bound(interval_image(g, p(k), p(k + 1)), interval_inf(g, p(k), p(k + 1)))
                is_set_lower_bound(interval_image(g, p(k), p(k + 1)), interval_inf(g, p(k), p(k + 1)))
                forall(v: Real) {
                    if interval_image(f, p(k), p(k + 1)).contains(v) {
                        interval_image(f, p(k), p(k + 1)).contains(v) = function_image_contains(f, interval_set(p(k), p(k + 1)), v)
                        function_image_contains(f, interval_set(p(k), p(k + 1)), v)
                        let t: Real satisfy {
                            interval_set(p(k), p(k + 1)).contains(t) and v = f(t)
                        }
                        interval_set(p(k), p(k + 1)).contains(t) = interval_contains(p(k), p(k + 1), t)
                        interval_contains(p(k), p(k + 1), t)
                        forall(t0: Real) {
                            interval_contains(p(k), p(k + 1), t0) implies g(t0) - eps <= f(t0)
                        }
                        interval_contains(p(k), p(k + 1), t) implies g(t) - eps <= f(t)
                        g(t) - eps <= f(t)
                        maps_into_set_image(interval_set(p(k), p(k + 1)), g, t)
                        function_image(g, interval_set(p(k), p(k + 1))).contains(g(t))
                        interval_image(g, p(k), p(k + 1)).contains(g(t))
                        set_lower_bound_contains_le(interval_image(g, p(k), p(k + 1)), interval_inf(g, p(k), p(k + 1)), g(t))
                        interval_inf(g, p(k), p(k + 1)) <= g(t)
                        add_le_add_right[Real](interval_inf(g, p(k), p(k + 1)), g(t), -eps)
                        interval_inf(g, p(k), p(k + 1)) + -eps <= g(t) + -eps
                        interval_inf(g, p(k), p(k + 1)) + -eps = interval_inf(g, p(k), p(k + 1)) - eps
                        g(t) + -eps = g(t) - eps
                        interval_inf(g, p(k), p(k + 1)) - eps <= g(t) - eps
                        lte_trans[Real](interval_inf(g, p(k), p(k + 1)) - eps, g(t) - eps, f(t))
                        interval_inf(g, p(k), p(k + 1)) - eps <= f(t)
                        v = f(t)
                        interval_inf(g, p(k), p(k + 1)) - eps <= v
                    }
                }
                is_set_lower_bound(interval_image(f, p(k), p(k + 1)), interval_inf(g, p(k), p(k + 1)) - eps)
                set_lower_bound_le_infimum(interval_image(f, p(k), p(k + 1)), interval_inf(f, p(k), p(k + 1)), interval_inf(g, p(k), p(k + 1)) - eps)
                interval_inf(g, p(k), p(k + 1)) - eps <= interval_inf(f, p(k), p(k + 1))
                p(k) <= p(k + 1)
                sub_nonneg(p(k), p(k + 1))
                Real.0 <= p(k + 1) - p(k)
                p(k + 1) - p(k) = diff_step(p, k)
                Real.0 <= diff_step(p, k)
                mul_le_mul_of_nonneg_right(interval_inf(g, p(k), p(k + 1)) - eps, interval_inf(f, p(k), p(k + 1)), diff_step(p, k))
                (interval_inf(g, p(k), p(k + 1)) - eps) * diff_step(p, k) <= interval_inf(f, p(k), p(k + 1)) * diff_step(p, k)
                mul_sub_distrib_left(interval_inf(g, p(k), p(k + 1)), eps, diff_step(p, k))
                (interval_inf(g, p(k), p(k + 1)) - eps) * diff_step(p, k) = interval_inf(g, p(k), p(k + 1)) * diff_step(p, k) - eps * diff_step(p, k)
                interval_inf(g, p(k), p(k + 1)) * diff_step(p, k) - eps * diff_step(p, k) <= interval_inf(f, p(k), p(k + 1)) * diff_step(p, k)
                partition_step_lower(g, p, k) - eps * diff_step(p, k) <= partition_step_lower(f, p, k)
                sub_seq(partition_step_lower(g, p), mul_fn(eps, diff_step(p)), k) = partition_step_lower(g, p, k) - mul_fn(eps, diff_step(p), k)
                mul_fn(eps, diff_step(p), k) = eps * diff_step(p, k)
                sub_seq(partition_step_lower(g, p), mul_fn(eps, diff_step(p)), k) <= partition_step_lower(f, p, k)
            }
        }
        partial_lte(sub_seq(partition_step_lower(g, p), mul_fn(eps, diff_step(p))), partition_step_lower(f, p), n)
        partial(sub_seq(partition_step_lower(g, p), mul_fn(eps, diff_step(p))), n) <= partial(partition_step_lower(f, p), n)
        partial_sub_seq(partition_step_lower(g, p), mul_fn(eps, diff_step(p)), n)
        partial(sub_seq(partition_step_lower(g, p), mul_fn(eps, diff_step(p))), n) = partial(partition_step_lower(g, p), n) - partial(mul_fn(eps, diff_step(p)), n)
        partial(partition_step_lower(g, p), n) = lower_sum(g, p, n)
        partial_scalar_mul[Real](eps, diff_step(p), n)
        partial(mul_fn(eps, diff_step(p)), n) = eps * partial(diff_step(p), n)
        telescope(p, n)
        partial(diff_step(p), n) = p(n) - p(Nat.0)
        partial(mul_fn(eps, diff_step(p)), n) = eps * (p(n) - p(Nat.0))
        partition_end(p, a, b, n)
        p(n) = b
        partition_start(p, a, b, n)
        p(Nat.0) = a
        partial(mul_fn(eps, diff_step(p)), n) = eps * (b - a)
        partial(sub_seq(partition_step_lower(g, p), mul_fn(eps, diff_step(p))), n) = lower_sum(g, p, n) - eps * (b - a)
        lower_sum(g, p, n) - eps * (b - a) <= lower_sum(f, p, n)
        lte_sub_imp_lte_add(lower_sum(g, p, n), eps * (b - a), lower_sum(f, p, n))
        lower_sum(g, p, n) <= lower_sum(f, p, n) + eps * (b - a)
    }
}

/// If f <= g + eps pointwise on [a, b], every upper sum of f is within
/// eps * (b - a) of the corresponding upper sum of g.
theorem upper_sum_eps_close(f: Real -> Real, g: Real -> Real, p: Nat -> Real, a: Real, b: Real, n: Nat, eps: Real, ub_f: Real, ub_g: Real) {
    is_partition(p, a, b, n) and
    (forall(t: Real) { interval_contains(a, b, t) implies f(t) <= ub_f }) and
    (forall(t: Real) { interval_contains(a, b, t) implies g(t) <= ub_g }) and
    (forall(t: Real) { interval_contains(a, b, t) implies f(t) <= g(t) + eps })
    implies
    upper_sum(f, p, n) <= upper_sum(g, p, n) + eps * (b - a)
} by {
    if is_partition(p, a, b, n) and
       (forall(t: Real) { interval_contains(a, b, t) implies f(t) <= ub_f }) and
       (forall(t: Real) { interval_contains(a, b, t) implies g(t) <= ub_g }) and
       (forall(t: Real) { interval_contains(a, b, t) implies f(t) <= g(t) + eps }) {
        forall(k: Nat) {
            if k < n {
                lt_imp_lte_suc(k, n)
                k + 1 <= n
                k <= k + 1
                partition_mono(p, a, b, n, k, k + 1)
                p(k) <= p(k + 1)
                partition_point_in_interval(p, a, b, n, k)
                interval_contains(a, b, p(k))
                partition_point_in_interval(p, a, b, n, k + 1)
                interval_contains(a, b, p(k + 1))
                forall(t: Real) {
                    if interval_contains(p(k), p(k + 1), t) {
                        interval_contains_left(p(k), p(k + 1), t)
                        p(k) <= t
                        interval_contains_right(p(k), p(k + 1), t)
                        t <= p(k + 1)
                        interval_contains_mono(a, b, p(k), p(k + 1), t)
                        interval_contains(a, b, t)
                        forall(t0: Real) {
                            interval_contains(a, b, t0) implies f(t0) <= ub_f
                        }
                        interval_contains(a, b, t) implies f(t) <= ub_f
                        f(t) <= ub_f
                    }
                }
                forall(t: Real) {
                    if interval_contains(p(k), p(k + 1), t) {
                        interval_contains_left(p(k), p(k + 1), t)
                        p(k) <= t
                        interval_contains_right(p(k), p(k + 1), t)
                        t <= p(k + 1)
                        interval_contains_mono(a, b, p(k), p(k + 1), t)
                        interval_contains(a, b, t)
                        forall(t1: Real) {
                            interval_contains(a, b, t1) implies g(t1) <= ub_g
                        }
                        interval_contains(a, b, t) implies g(t) <= ub_g
                        g(t) <= ub_g
                    }
                }
                forall(t: Real) {
                    if interval_contains(p(k), p(k + 1), t) {
                        interval_contains_left(p(k), p(k + 1), t)
                        p(k) <= t
                        interval_contains_right(p(k), p(k + 1), t)
                        t <= p(k + 1)
                        interval_contains_mono(a, b, p(k), p(k + 1), t)
                        interval_contains(a, b, t)
                        forall(t2: Real) {
                            interval_contains(a, b, t2) implies f(t2) <= g(t2) + eps
                        }
                        interval_contains(a, b, t) implies f(t) <= g(t) + eps
                        f(t) <= g(t) + eps
                    }
                }
                forall(t: Real) {
                    interval_contains(p(k), p(k + 1), t) implies f(t) <= ub_f
                }
                forall(t: Real) {
                    interval_contains(p(k), p(k + 1), t) implies g(t) <= ub_g
                }
                forall(t: Real) {
                    interval_contains(p(k), p(k + 1), t) implies f(t) <= g(t) + eps
                }
                interval_set_contains_left(p(k), p(k + 1))
                is_nonempty(interval_set(p(k), p(k + 1)))
                image_upper_bound(f, p(k), p(k + 1), ub_f)
                is_set_upper_bound(interval_image(f, p(k), p(k + 1)), ub_f)
                exists(b1: Real) {
                    is_set_upper_bound(interval_image(f, p(k), p(k + 1)), b1)
                }
                has_upper_bound(interval_image(f, p(k), p(k + 1)))
                is_nonempty(interval_set(p(k), p(k + 1))) and has_upper_bound(interval_image(f, p(k), p(k + 1)))
                interval_sup_spec(f, p(k), p(k + 1))
                is_set_supremum(interval_image(f, p(k), p(k + 1)), interval_sup(f, p(k), p(k + 1)))
                image_upper_bound(g, p(k), p(k + 1), ub_g)
                is_set_upper_bound(interval_image(g, p(k), p(k + 1)), ub_g)
                exists(b2: Real) {
                    is_set_upper_bound(interval_image(g, p(k), p(k + 1)), b2)
                }
                has_upper_bound(interval_image(g, p(k), p(k + 1)))
                is_nonempty(interval_set(p(k), p(k + 1))) and has_upper_bound(interval_image(g, p(k), p(k + 1)))
                interval_sup_spec(g, p(k), p(k + 1))
                is_set_supremum(interval_image(g, p(k), p(k + 1)), interval_sup(g, p(k), p(k + 1)))
                set_supremum_is_upper_bound(interval_image(g, p(k), p(k + 1)), interval_sup(g, p(k), p(k + 1)))
                is_set_upper_bound(interval_image(g, p(k), p(k + 1)), interval_sup(g, p(k), p(k + 1)))
                forall(v: Real) {
                    if interval_image(f, p(k), p(k + 1)).contains(v) {
                        interval_image(f, p(k), p(k + 1)).contains(v) = function_image_contains(f, interval_set(p(k), p(k + 1)), v)
                        function_image_contains(f, interval_set(p(k), p(k + 1)), v)
                        let t: Real satisfy {
                            interval_set(p(k), p(k + 1)).contains(t) and v = f(t)
                        }
                        interval_set(p(k), p(k + 1)).contains(t) = interval_contains(p(k), p(k + 1), t)
                        interval_contains(p(k), p(k + 1), t)
                        forall(t0: Real) {
                            interval_contains(p(k), p(k + 1), t0) implies f(t0) <= g(t0) + eps
                        }
                        interval_contains(p(k), p(k + 1), t) implies f(t) <= g(t) + eps
                        f(t) <= g(t) + eps
                        maps_into_set_image(interval_set(p(k), p(k + 1)), g, t)
                        function_image(g, interval_set(p(k), p(k + 1))).contains(g(t))
                        interval_image(g, p(k), p(k + 1)).contains(g(t))
                        set_upper_bound_contains_le(interval_image(g, p(k), p(k + 1)), interval_sup(g, p(k), p(k + 1)), g(t))
                        g(t) <= interval_sup(g, p(k), p(k + 1))
                        add_le_add_right[Real](g(t), interval_sup(g, p(k), p(k + 1)), eps)
                        g(t) + eps <= interval_sup(g, p(k), p(k + 1)) + eps
                        lte_trans[Real](f(t), g(t) + eps, interval_sup(g, p(k), p(k + 1)) + eps)
                        f(t) <= interval_sup(g, p(k), p(k + 1)) + eps
                        v = f(t)
                        v <= interval_sup(g, p(k), p(k + 1)) + eps
                    }
                }
                is_set_upper_bound(interval_image(f, p(k), p(k + 1)), interval_sup(g, p(k), p(k + 1)) + eps)
                set_supremum_le_upper_bound(interval_image(f, p(k), p(k + 1)), interval_sup(f, p(k), p(k + 1)), interval_sup(g, p(k), p(k + 1)) + eps)
                interval_sup(f, p(k), p(k + 1)) <= interval_sup(g, p(k), p(k + 1)) + eps
                lte_imp_lte_sub(interval_sup(f, p(k), p(k + 1)), interval_sup(g, p(k), p(k + 1)), eps)
                interval_sup(f, p(k), p(k + 1)) - eps <= interval_sup(g, p(k), p(k + 1))
                p(k) <= p(k + 1)
                sub_nonneg(p(k), p(k + 1))
                Real.0 <= p(k + 1) - p(k)
                p(k + 1) - p(k) = diff_step(p, k)
                Real.0 <= diff_step(p, k)
                mul_le_mul_of_nonneg_right(interval_sup(f, p(k), p(k + 1)) - eps, interval_sup(g, p(k), p(k + 1)), diff_step(p, k))
                (interval_sup(f, p(k), p(k + 1)) - eps) * diff_step(p, k) <= interval_sup(g, p(k), p(k + 1)) * diff_step(p, k)
                mul_sub_distrib_left(interval_sup(f, p(k), p(k + 1)), eps, diff_step(p, k))
                (interval_sup(f, p(k), p(k + 1)) - eps) * diff_step(p, k) = interval_sup(f, p(k), p(k + 1)) * diff_step(p, k) - eps * diff_step(p, k)
                interval_sup(f, p(k), p(k + 1)) * diff_step(p, k) - eps * diff_step(p, k) <= interval_sup(g, p(k), p(k + 1)) * diff_step(p, k)
                partition_step_upper(f, p, k) - eps * diff_step(p, k) <= partition_step_upper(g, p, k)
                sub_seq(partition_step_upper(f, p), mul_fn(eps, diff_step(p)), k) = partition_step_upper(f, p, k) - mul_fn(eps, diff_step(p), k)
                mul_fn(eps, diff_step(p), k) = eps * diff_step(p, k)
                sub_seq(partition_step_upper(f, p), mul_fn(eps, diff_step(p)), k) <= partition_step_upper(g, p, k)
            }
        }
        partial_lte(sub_seq(partition_step_upper(f, p), mul_fn(eps, diff_step(p))), partition_step_upper(g, p), n)
        partial(sub_seq(partition_step_upper(f, p), mul_fn(eps, diff_step(p))), n) <= partial(partition_step_upper(g, p), n)
        partial_sub_seq(partition_step_upper(f, p), mul_fn(eps, diff_step(p)), n)
        partial(sub_seq(partition_step_upper(f, p), mul_fn(eps, diff_step(p))), n) = partial(partition_step_upper(f, p), n) - partial(mul_fn(eps, diff_step(p)), n)
        partial(partition_step_upper(f, p), n) = upper_sum(f, p, n)
        partial_scalar_mul[Real](eps, diff_step(p), n)
        partial(mul_fn(eps, diff_step(p)), n) = eps * partial(diff_step(p), n)
        telescope(p, n)
        partial(diff_step(p), n) = p(n) - p(Nat.0)
        partial(mul_fn(eps, diff_step(p)), n) = eps * (p(n) - p(Nat.0))
        partition_end(p, a, b, n)
        p(n) = b
        partition_start(p, a, b, n)
        p(Nat.0) = a
        partial(mul_fn(eps, diff_step(p)), n) = eps * (b - a)
        partial(sub_seq(partition_step_upper(f, p), mul_fn(eps, diff_step(p))), n) = upper_sum(f, p, n) - eps * (b - a)
        upper_sum(f, p, n) - eps * (b - a) <= upper_sum(g, p, n)
        lte_sub_imp_lte_add(upper_sum(f, p, n), eps * (b - a), upper_sum(g, p, n))
        upper_sum(f, p, n) <= upper_sum(g, p, n) + eps * (b - a)
    }
}

// ---------------------------------------------------------------------------
// Interchange of the uniform limit and the Riemann integral
// ---------------------------------------------------------------------------
//
// If f_n converges to g uniformly on [a, b], each f_n is integrable on [a, b]
// and so is g, and the whole sequence is bounded between lb and ub on [a, b],
// then integral(f_n, a, b) converges to integral(g, a, b).  The eps-sandwich:
// pointwise |f_n(x) - g(x)| < eps3 on [a, b] keeps every lower sum of g within
// eps3 * (b - a) of the corresponding lower sum of f_n, and every upper sum of
// f_n within eps3 * (b - a) of the corresponding upper sum of g (the sums
// differ by at most eps3 per cell, times the cell width).  Passing to the
// supremum of the lower sums and the infimum of the upper sums bounds
// |integral(f_n) - integral(g)| by eps3 * (b - a), which the helper
// exists_eps_mul_lt shrinks below the required tolerance eps.

/// Pointwise eps-closeness of f_n to g on [a, b] gives the lower eps-slack
/// bound g(t) - eps <= f(n, t).
theorem uniform_eps_slack_lower(f: Nat -> Real -> Real, a: Real, b: Real, g: Real -> Real, n: Nat, eps: Real) {
    (forall(x: Real) { interval_contains(a, b, x) implies f(n, x).is_close(g(x), eps) })
    implies
    (forall(t: Real) { interval_contains(a, b, t) implies g(t) - eps <= f(n, t) })
} by {
    if forall(x: Real) { interval_contains(a, b, x) implies f(n, x).is_close(g(x), eps) } {
        forall(t: Real) {
            if interval_contains(a, b, t) {
                forall(x: Real) {
                    interval_contains(a, b, x) implies f(n, x).is_close(g(x), eps)
                }
                interval_contains(a, b, t) implies f(n, t).is_close(g(t), eps)
                f(n, t).is_close(g(t), eps)
                close_imp_bounds(f(n, t), g(t), eps)
                g(t) - eps < f(n, t) and f(n, t) < g(t) + eps and f(n, t) > g(t) - eps and g(t) < f(n, t) + eps
                g(t) - eps < f(n, t)
                lt_imp_lte(g(t) - eps, f(n, t))
                g(t) - eps <= f(n, t)
            }
        }
    }
}

/// Pointwise eps-closeness of f_n to g on [a, b] gives the upper eps-slack
/// bound f(n, t) <= g(t) + eps.
theorem uniform_eps_slack_upper(f: Nat -> Real -> Real, a: Real, b: Real, g: Real -> Real, n: Nat, eps: Real) {
    (forall(x: Real) { interval_contains(a, b, x) implies f(n, x).is_close(g(x), eps) })
    implies
    (forall(t: Real) { interval_contains(a, b, t) implies f(n, t) <= g(t) + eps })
} by {
    if forall(x: Real) { interval_contains(a, b, x) implies f(n, x).is_close(g(x), eps) } {
        forall(t: Real) {
            if interval_contains(a, b, t) {
                forall(x: Real) {
                    interval_contains(a, b, x) implies f(n, x).is_close(g(x), eps)
                }
                interval_contains(a, b, t) implies f(n, t).is_close(g(t), eps)
                f(n, t).is_close(g(t), eps)
                close_imp_bounds(f(n, t), g(t), eps)
                g(t) - eps < f(n, t) and f(n, t) < g(t) + eps and f(n, t) > g(t) - eps and g(t) < f(n, t) + eps
                f(n, t) < g(t) + eps
                lt_imp_lte(f(n, t), g(t) + eps)
                f(n, t) <= g(t) + eps
            }
        }
    }
}

/// If f_n converges to g uniformly on [a, b], each f_n is integrable on
/// [a, b] and so is g, and the sequence is uniformly bounded between lb and
/// ub, then the integrals of the f_n converge to the integral of g.
theorem uniform_converges_integral_interchange(f: Nat -> Real -> Real, a: Real, b: Real, g: Real -> Real, lb: Real, ub: Real) {
    a <= b and
    uniform_converges_on(f, a, b, g) and
    is_integrable(g, a, b) and
    (forall(n: Nat) { is_integrable(f(n), a, b) }) and
    (forall(n: Nat, t: Real) { interval_contains(a, b, t) implies lb <= f(n, t) }) and
    (forall(n: Nat, t: Real) { interval_contains(a, b, t) implies f(n, t) <= ub }) and
    (forall(t: Real) { interval_contains(a, b, t) implies lb <= g(t) }) and
    (forall(t: Real) { interval_contains(a, b, t) implies g(t) <= ub })
    implies
    converges_to(integral_sequence(f, a, b), integral(g, a, b))
} by {
    if a <= b and
       uniform_converges_on(f, a, b, g) and
       is_integrable(g, a, b) and
       (forall(n: Nat) { is_integrable(f(n), a, b) }) and
       (forall(n: Nat, t: Real) { interval_contains(a, b, t) implies lb <= f(n, t) }) and
       (forall(n: Nat, t: Real) { interval_contains(a, b, t) implies f(n, t) <= ub }) and
       (forall(t: Real) { interval_contains(a, b, t) implies lb <= g(t) }) and
       (forall(t: Real) { interval_contains(a, b, t) implies g(t) <= ub }) {
        converges_to(integral_sequence(f, a, b), integral(g, a, b)) = forall(eps: Real) {
            eps.is_positive implies exists(n: Nat) {
                tail_bound(integral_sequence(f, a, b), integral(g, a, b), n, eps)
            }
        }
        forall(eps: Real) {
            if eps.is_positive {
                sub_nonneg(a, b)
                Real.0 <= b - a
                exists_eps_mul_lt(b - a, eps)
                exists(eps3: Real) {
                    eps3.is_positive and eps3 * (b - a) < eps
                }
                let eps3: Real satisfy {
                    eps3.is_positive and eps3 * (b - a) < eps
                }
                eps3.is_positive
                eps3 * (b - a) < eps
                uniform_converges_on_apply(f, a, b, g, eps3)
                exists(n0: Nat) {
                    forall(i: Nat) {
                        n0 <= i implies forall(x: Real) {
                            interval_contains(a, b, x) implies f(i, x).is_close(g(x), eps3)
                        }
                    }
                }
                let n0: Nat satisfy {
                    forall(i: Nat) {
                        n0 <= i implies forall(x: Real) {
                            interval_contains(a, b, x) implies f(i, x).is_close(g(x), eps3)
                        }
                    }
                }
                tail_bound(integral_sequence(f, a, b), integral(g, a, b), n0, eps) = forall(i: Nat) {
                    n0 <= i implies integral_sequence(f, a, b, i).is_close(integral(g, a, b), eps)
                }
                forall(i: Nat) {
                    if n0 <= i {
                        forall(x: Real) {
                            interval_contains(a, b, x) implies f(i, x).is_close(g(x), eps3)
                        }
                        uniform_eps_slack_lower(f, a, b, g, i, eps3)
                        forall(t: Real) {
                            interval_contains(a, b, t) implies g(t) - eps3 <= f(i, t)
                        }
                        uniform_eps_slack_upper(f, a, b, g, i, eps3)
                        forall(t: Real) {
                            interval_contains(a, b, t) implies f(i, t) <= g(t) + eps3
                        }
                        forall(t: Real) {
                            interval_contains(a, b, t) implies lb <= f(i, t)
                        }
                        forall(t: Real) {
                            interval_contains(a, b, t) implies f(i, t) <= ub
                        }
                        forall(t: Real) {
                            interval_contains(a, b, t) implies lb <= g(t)
                        }
                        forall(t: Real) {
                            interval_contains(a, b, t) implies g(t) <= ub
                        }
                        is_integrable(g, a, b)
                        integral_spec(g, a, b)
                        is_set_supremum(lower_sum_set(g, a, b), integral(g, a, b)) and is_set_infimum(upper_sum_set(g, a, b), integral(g, a, b))
                        is_set_supremum(lower_sum_set(g, a, b), integral(g, a, b))
                        is_set_infimum(upper_sum_set(g, a, b), integral(g, a, b))
                        forall(n: Nat) {
                            is_integrable(f(n), a, b)
                        }
                        is_integrable(f(i), a, b)
                        integral_spec(f(i), a, b)
                        is_set_supremum(lower_sum_set(f(i), a, b), integral(f(i), a, b)) and is_set_infimum(upper_sum_set(f(i), a, b), integral(f(i), a, b))
                        is_set_supremum(lower_sum_set(f(i), a, b), integral(f(i), a, b))
                        is_set_infimum(upper_sum_set(f(i), a, b), integral(f(i), a, b))
                        forall(v: Real) {
                            if lower_sum_set(g, a, b).contains(v) {
                                lower_sum_set(g, a, b).contains(v) = lower_sum_contains(g, a, b, v)
                                lower_sum_contains(g, a, b, v)
                                lower_sum_contains(g, a, b, v) = exists(p: Nat -> Real, n: Nat) {
                                    is_partition(p, a, b, n) and v = lower_sum(g, p, n)
                                }
                                let (p: Nat -> Real, n: Nat) satisfy {
                                    is_partition(p, a, b, n) and v = lower_sum(g, p, n)
                                }
                                is_partition(p, a, b, n)
                                v = lower_sum(g, p, n)
                                lower_sum_eps_close(f(i), g, p, a, b, n, eps3, lb, lb)
                                lower_sum(g, p, n) <= lower_sum(f(i), p, n) + eps3 * (b - a)
                                is_partition(p, a, b, n) and lower_sum(f(i), p, n) = lower_sum(f(i), p, n)
                                exists(p1: Nat -> Real, n1: Nat) {
                                    is_partition(p1, a, b, n1) and lower_sum(f(i), p, n) = lower_sum(f(i), p1, n1)
                                }
                                lower_sum_contains(f(i), a, b, lower_sum(f(i), p, n))
                                lower_sum_set(f(i), a, b).contains(lower_sum(f(i), p, n)) = lower_sum_contains(f(i), a, b, lower_sum(f(i), p, n))
                                lower_sum_set(f(i), a, b).contains(lower_sum(f(i), p, n))
                                set_member_le_supremum(lower_sum_set(f(i), a, b), integral(f(i), a, b), lower_sum(f(i), p, n))
                                lower_sum(f(i), p, n) <= integral(f(i), a, b)
                                add_le_add_right[Real](lower_sum(f(i), p, n), integral(f(i), a, b), eps3 * (b - a))
                                lower_sum(f(i), p, n) + eps3 * (b - a) <= integral(f(i), a, b) + eps3 * (b - a)
                                lte_trans[Real](lower_sum(g, p, n), lower_sum(f(i), p, n) + eps3 * (b - a), integral(f(i), a, b) + eps3 * (b - a))
                                lower_sum(g, p, n) <= integral(f(i), a, b) + eps3 * (b - a)
                                v = lower_sum(g, p, n)
                                v <= integral(f(i), a, b) + eps3 * (b - a)
                            }
                        }
                        is_set_upper_bound(lower_sum_set(g, a, b), integral(f(i), a, b) + eps3 * (b - a))
                        set_supremum_le_upper_bound(lower_sum_set(g, a, b), integral(g, a, b), integral(f(i), a, b) + eps3 * (b - a))
                        integral(g, a, b) <= integral(f(i), a, b) + eps3 * (b - a)
                        forall(v: Real) {
                            if upper_sum_set(g, a, b).contains(v) {
                                upper_sum_set(g, a, b).contains(v) = upper_sum_contains(g, a, b, v)
                                upper_sum_contains(g, a, b, v)
                                upper_sum_contains(g, a, b, v) = exists(p: Nat -> Real, n: Nat) {
                                    is_partition(p, a, b, n) and v = upper_sum(g, p, n)
                                }
                                let (p: Nat -> Real, n: Nat) satisfy {
                                    is_partition(p, a, b, n) and v = upper_sum(g, p, n)
                                }
                                is_partition(p, a, b, n)
                                v = upper_sum(g, p, n)
                                upper_sum_eps_close(f(i), g, p, a, b, n, eps3, ub, ub)
                                upper_sum(f(i), p, n) <= upper_sum(g, p, n) + eps3 * (b - a)
                                is_partition(p, a, b, n) and upper_sum(f(i), p, n) = upper_sum(f(i), p, n)
                                exists(p2: Nat -> Real, n2: Nat) {
                                    is_partition(p2, a, b, n2) and upper_sum(f(i), p, n) = upper_sum(f(i), p2, n2)
                                }
                                upper_sum_contains(f(i), a, b, upper_sum(f(i), p, n))
                                upper_sum_set(f(i), a, b).contains(upper_sum(f(i), p, n)) = upper_sum_contains(f(i), a, b, upper_sum(f(i), p, n))
                                upper_sum_set(f(i), a, b).contains(upper_sum(f(i), p, n))
                                set_infimum_is_lower_bound(upper_sum_set(f(i), a, b), integral(f(i), a, b))
                                is_set_lower_bound(upper_sum_set(f(i), a, b), integral(f(i), a, b))
                                set_lower_bound_contains_le(upper_sum_set(f(i), a, b), integral(f(i), a, b), upper_sum(f(i), p, n))
                                integral(f(i), a, b) <= upper_sum(f(i), p, n)
                                lte_imp_lte_sub(upper_sum(f(i), p, n), upper_sum(g, p, n), eps3 * (b - a))
                                upper_sum(f(i), p, n) - eps3 * (b - a) <= upper_sum(g, p, n)
                                add_le_add_right[Real](integral(f(i), a, b), upper_sum(f(i), p, n), -(eps3 * (b - a)))
                                integral(f(i), a, b) + -(eps3 * (b - a)) <= upper_sum(f(i), p, n) + -(eps3 * (b - a))
                                integral(f(i), a, b) + -(eps3 * (b - a)) = integral(f(i), a, b) - eps3 * (b - a)
                                upper_sum(f(i), p, n) + -(eps3 * (b - a)) = upper_sum(f(i), p, n) - eps3 * (b - a)
                                integral(f(i), a, b) - eps3 * (b - a) <= upper_sum(f(i), p, n) - eps3 * (b - a)
                                lte_trans[Real](integral(f(i), a, b) - eps3 * (b - a), upper_sum(f(i), p, n) - eps3 * (b - a), upper_sum(g, p, n))
                                integral(f(i), a, b) - eps3 * (b - a) <= upper_sum(g, p, n)
                                v = upper_sum(g, p, n)
                                integral(f(i), a, b) - eps3 * (b - a) <= v
                            }
                        }
                        is_set_lower_bound(upper_sum_set(g, a, b), integral(f(i), a, b) - eps3 * (b - a))
                        set_lower_bound_le_infimum(upper_sum_set(g, a, b), integral(g, a, b), integral(f(i), a, b) - eps3 * (b - a))
                        integral(f(i), a, b) - eps3 * (b - a) <= integral(g, a, b)
                        lte_sub_imp_lte_add(integral(f(i), a, b), eps3 * (b - a), integral(g, a, b))
                        integral(f(i), a, b) <= integral(g, a, b) + eps3 * (b - a)
                        lt_add_right(eps3 * (b - a), eps, integral(g, a, b))
                        eps3 * (b - a) + integral(g, a, b) < eps + integral(g, a, b)
                        add_comm(eps3 * (b - a), integral(g, a, b))
                        eps3 * (b - a) + integral(g, a, b) = integral(g, a, b) + eps3 * (b - a)
                        add_comm(eps, integral(g, a, b))
                        eps + integral(g, a, b) = integral(g, a, b) + eps
                        integral(g, a, b) + eps3 * (b - a) < integral(g, a, b) + eps
                        lt_of_lte_of_lt(integral(f(i), a, b), integral(g, a, b) + eps3 * (b - a), integral(g, a, b) + eps)
                        integral(f(i), a, b) < integral(g, a, b) + eps
                        lt_add_right(eps3 * (b - a), eps, integral(f(i), a, b))
                        eps3 * (b - a) + integral(f(i), a, b) < eps + integral(f(i), a, b)
                        add_comm(eps3 * (b - a), integral(f(i), a, b))
                        eps3 * (b - a) + integral(f(i), a, b) = integral(f(i), a, b) + eps3 * (b - a)
                        add_comm(eps, integral(f(i), a, b))
                        eps + integral(f(i), a, b) = integral(f(i), a, b) + eps
                        integral(f(i), a, b) + eps3 * (b - a) < integral(f(i), a, b) + eps
                        lt_of_lte_of_lt(integral(g, a, b), integral(f(i), a, b) + eps3 * (b - a), integral(f(i), a, b) + eps)
                        integral(g, a, b) < integral(f(i), a, b) + eps
                        lt_add_right(integral(g, a, b), integral(f(i), a, b) + eps, -eps)
                        integral(g, a, b) + -eps < integral(f(i), a, b) + eps + -eps
                        integral(g, a, b) + -eps = integral(g, a, b) - eps
                        add_assoc(integral(f(i), a, b), eps, -eps)
                        (integral(f(i), a, b) + eps) + -eps = integral(f(i), a, b) + (eps + -eps)
                        add_neg_eq_zero(eps)
                        eps + -eps = Real.0
                        integral(f(i), a, b) + (eps + -eps) = integral(f(i), a, b) + Real.0
                        add_zero_right(integral(f(i), a, b))
                        integral(f(i), a, b) + Real.0 = integral(f(i), a, b)
                        (integral(f(i), a, b) + eps) + -eps = integral(f(i), a, b)
                        integral(g, a, b) - eps < integral(f(i), a, b)
                        bounds_imp_close(integral(f(i), a, b), integral(g, a, b), eps)
                        integral(f(i), a, b).is_close(integral(g, a, b), eps)
                        integral_sequence(f, a, b, i) = integral(f(i), a, b)
                        integral_sequence(f, a, b, i).is_close(integral(g, a, b), eps)
                    }
                }
                tail_bound(integral_sequence(f, a, b), integral(g, a, b), n0, eps)
                exists(n: Nat) {
                    tail_bound(integral_sequence(f, a, b), integral(g, a, b), n, eps)
                }
            }
        }
    }
}
