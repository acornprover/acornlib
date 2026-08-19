from data.basic.functions import identity_fn
from order import lte_trans, lt_imp_lte, lt_of_lt_of_lte
from order_set import closed_interval_set, closed_interval_set_lower_le
from real.continuity_base import Real, uniform, uniform_condition
from real.continuity_sequences import identity_function_is_uniform
from real.real_base import abs_neg, abs_gte_zero, pos_imp_eq_abs, pos_gt_zero, gt_zero_imp_pos, lte_lt_trans
from real.real_field import div_le_of_mul_le, div_mul_cancel_left, mul_div_cancel
from real.real_ring import mul_pos_pos
from real.exp import abs_div
from real.derivative_quotient import reciprocal_real, reciprocal_real_apply, inverse_sub_inverse
from algebra.field.field import mul_not_zero
from ordered_field import mul_le_mul_of_nonneg_right

numerals Real

/// Substituting an equation into a negated proposition: p = q and not p gives not q.
theorem eq_neg_transport(p: Bool, q: Bool) {
    p = q and not p implies not q
} by {
    if p = q and not p {
        not p
        p = q
        not q
    }
}

/// Dividing both sides of an inequality by a positive number preserves it.
theorem div_le_div_pos(a: Real, b: Real, c: Real) {
    a <= b and Real.0 < c implies a / c <= b / c
} by {
    if a <= b and Real.0 < c {
        c != Real.0
        mul_div_cancel(a, c)
        c * (a / c) = a
        (a / c) * c = c * (a / c)
        (a / c) * c = a
        (a / c) * c <= b
        div_le_of_mul_le(a / c, c, b)
        a / c <= b / c
    }
}

/// True if f satisfies the delta-epsilon condition for uniform continuity on the closed interval [lower, upper].
define uniform_on_condition(f: Real -> Real, lower: Real, upper: Real, delta: Real, eps: Real) -> Bool {
    forall(x: Real, y: Real) {
        closed_interval_set(lower, upper).contains(x) and
        closed_interval_set(lower, upper).contains(y) and
        x.is_close(y, delta)
        implies f(x).is_close(f(y), eps)
    }
}

/// True if f is uniformly continuous on the closed interval [lower, upper].
define uniform_on(f: Real -> Real, lower: Real, upper: Real) -> Bool {
    forall(eps: Real) {
        eps.is_positive implies exists(delta: Real) {
            delta.is_positive and uniform_on_condition(f, lower, upper, delta, eps)
        }
    }
}

/// True if f satisfies the delta-epsilon condition for continuity at x relative to [lower, upper].
define continuous_on_condition(f: Real -> Real, lower: Real, upper: Real, x: Real, delta: Real, eps: Real) -> Bool {
    forall(x1: Real) {
        closed_interval_set(lower, upper).contains(x1) and x1.is_close(x, delta)
        implies f(x1).is_close(f(x), eps)
    }
}

/// True if f is continuous at the point x relative to the closed interval [lower, upper].
define continuous_on_at(f: Real -> Real, lower: Real, upper: Real, x: Real) -> Bool {
    forall(eps: Real) {
        eps.is_positive implies exists(delta: Real) {
            delta.is_positive and continuous_on_condition(f, lower, upper, x, delta, eps)
        }
    }
}

/// True if f is continuous at every point of the closed interval [lower, upper].
define continuous_on(f: Real -> Real, lower: Real, upper: Real) -> Bool {
    forall(x: Real) {
        closed_interval_set(lower, upper).contains(x) implies continuous_on_at(f, lower, upper, x)
    }
}

/// Uniform continuity on [lower, upper] implies continuity at every point of the interval.
theorem uniform_on_imp_continuous_on(f: Real -> Real, lower: Real, upper: Real) {
    uniform_on(f, lower, upper)
    implies
    continuous_on(f, lower, upper)
} by {
    forall(x: Real) {
        if closed_interval_set(lower, upper).contains(x) {
            forall(eps: Real) {
                if eps.is_positive {
                    uniform_on(f, lower, upper) = forall(eps2: Real) {
                        eps2.is_positive implies exists(delta2: Real) {
                            delta2.is_positive and uniform_on_condition(f, lower, upper, delta2, eps2)
                        }
                    }
                    eps.is_positive implies exists(delta2: Real) {
                        delta2.is_positive and uniform_on_condition(f, lower, upper, delta2, eps)
                    }
                    exists(delta2: Real) {
                        delta2.is_positive and uniform_on_condition(f, lower, upper, delta2, eps)
                    }
                    let delta: Real satisfy {
                        delta.is_positive and uniform_on_condition(f, lower, upper, delta, eps)
                    }
                    forall(x1: Real) {
                        if closed_interval_set(lower, upper).contains(x1) and x1.is_close(x, delta) {
                            closed_interval_set(lower, upper).contains(x)
                            uniform_on_condition(f, lower, upper, delta, eps) = forall(x2: Real, y2: Real) {
                                closed_interval_set(lower, upper).contains(x2) and
                                closed_interval_set(lower, upper).contains(y2) and
                                x2.is_close(y2, delta)
                                implies f(x2).is_close(f(y2), eps)
                            }
                            f(x1).is_close(f(x), eps)
                        }
                    }
                    continuous_on_condition(f, lower, upper, x, delta, eps)
                    delta.is_positive and continuous_on_condition(f, lower, upper, x, delta, eps)
                    exists(delta2: Real) {
                        delta2.is_positive and continuous_on_condition(f, lower, upper, x, delta2, eps)
                    }
                }
            }
            if not continuous_on_at(f, lower, upper, x) {
                continuous_on_at(f, lower, upper, x) = forall(eps2: Real) {
                    eps2.is_positive implies exists(delta2: Real) {
                        delta2.is_positive and continuous_on_condition(f, lower, upper, x, delta2, eps2)
                    }
                }
                eq_neg_transport(continuous_on_at(f, lower, upper, x), forall(eps2: Real) {
                    eps2.is_positive implies exists(delta2: Real) {
                        delta2.is_positive and continuous_on_condition(f, lower, upper, x, delta2, eps2)
                    }
                })
                not forall(eps2: Real) {
                    eps2.is_positive implies exists(delta2: Real) {
                        delta2.is_positive and continuous_on_condition(f, lower, upper, x, delta2, eps2)
                    }
                }
                exists(eps0: Real) {
                    eps0.is_positive and forall(delta0: Real) {
                        not delta0.is_positive or not continuous_on_condition(f, lower, upper, x, delta0, eps0)
                    }
                }
                let eps0: Real satisfy {
                    eps0.is_positive and forall(delta0: Real) {
                        not delta0.is_positive or not continuous_on_condition(f, lower, upper, x, delta0, eps0)
                    }
                }
                eps0.is_positive implies exists(delta0: Real) {
                    delta0.is_positive and continuous_on_condition(f, lower, upper, x, delta0, eps0)
                }
                exists(delta0: Real) {
                    delta0.is_positive and continuous_on_condition(f, lower, upper, x, delta0, eps0)
                }
                let delta0: Real satisfy {
                    delta0.is_positive and continuous_on_condition(f, lower, upper, x, delta0, eps0)
                }
                delta0.is_positive
                not delta0.is_positive or not continuous_on_condition(f, lower, upper, x, delta0, eps0)
                not continuous_on_condition(f, lower, upper, x, delta0, eps0)
                continuous_on_condition(f, lower, upper, x, delta0, eps0)
                false
            }
            continuous_on_at(f, lower, upper, x)
        }
    }
    if not continuous_on(f, lower, upper) {
        continuous_on(f, lower, upper) = forall(x0: Real) {
            closed_interval_set(lower, upper).contains(x0) implies continuous_on_at(f, lower, upper, x0)
        }
        eq_neg_transport(continuous_on(f, lower, upper), forall(x0: Real) {
            closed_interval_set(lower, upper).contains(x0) implies continuous_on_at(f, lower, upper, x0)
        })
        not forall(x0: Real) {
            closed_interval_set(lower, upper).contains(x0) implies continuous_on_at(f, lower, upper, x0)
        }
        exists(x0: Real) {
            closed_interval_set(lower, upper).contains(x0) and not continuous_on_at(f, lower, upper, x0)
        }
        let x0: Real satisfy {
            closed_interval_set(lower, upper).contains(x0) and not continuous_on_at(f, lower, upper, x0)
        }
        closed_interval_set(lower, upper).contains(x0) implies continuous_on_at(f, lower, upper, x0)
        continuous_on_at(f, lower, upper, x0)
        not continuous_on_at(f, lower, upper, x0)
        false
    }
    continuous_on(f, lower, upper)
}

/// Uniform continuity on the whole real line implies uniform continuity on any closed interval.
theorem uniform_imp_uniform_on(f: Real -> Real, lower: Real, upper: Real) {
    uniform(f)
    implies
    uniform_on(f, lower, upper)
} by {
    forall(eps: Real) {
        if eps.is_positive {
            uniform(f) = forall(eps2: Real) {
                eps2.is_positive implies exists(delta2: Real) {
                    delta2.is_positive and uniform_condition(f, delta2, eps2)
                }
            }
            eps.is_positive implies exists(delta2: Real) {
                delta2.is_positive and uniform_condition(f, delta2, eps)
            }
            exists(delta2: Real) {
                delta2.is_positive and uniform_condition(f, delta2, eps)
            }
            let delta: Real satisfy {
                delta.is_positive and uniform_condition(f, delta, eps)
            }
            forall(x: Real, y: Real) {
                if closed_interval_set(lower, upper).contains(x) and
                   closed_interval_set(lower, upper).contains(y) and
                   x.is_close(y, delta) {
                    uniform_condition(f, delta, eps) = forall(r1: Real, r2: Real) {
                        r1.is_close(r2, delta) implies f(r1).is_close(f(r2), eps)
                    }
                    f(x).is_close(f(y), eps)
                }
            }
            uniform_on_condition(f, lower, upper, delta, eps)
            delta.is_positive and uniform_on_condition(f, lower, upper, delta, eps)
            exists(delta2: Real) {
                delta2.is_positive and uniform_on_condition(f, lower, upper, delta2, eps)
            }
        }
    }
    if not uniform_on(f, lower, upper) {
        uniform_on(f, lower, upper) = forall(eps2: Real) {
            eps2.is_positive implies exists(delta2: Real) {
                delta2.is_positive and uniform_on_condition(f, lower, upper, delta2, eps2)
            }
        }
        eq_neg_transport(uniform_on(f, lower, upper), forall(eps2: Real) {
            eps2.is_positive implies exists(delta2: Real) {
                delta2.is_positive and uniform_on_condition(f, lower, upper, delta2, eps2)
            }
        })
        not forall(eps2: Real) {
            eps2.is_positive implies exists(delta2: Real) {
                delta2.is_positive and uniform_on_condition(f, lower, upper, delta2, eps2)
            }
        }
        exists(eps0: Real) {
            eps0.is_positive and forall(delta0: Real) {
                not delta0.is_positive or not uniform_on_condition(f, lower, upper, delta0, eps0)
            }
        }
        let eps0: Real satisfy {
            eps0.is_positive and forall(delta0: Real) {
                not delta0.is_positive or not uniform_on_condition(f, lower, upper, delta0, eps0)
            }
        }
        eps0.is_positive implies exists(delta0: Real) {
            delta0.is_positive and uniform_on_condition(f, lower, upper, delta0, eps0)
        }
        exists(delta0: Real) {
            delta0.is_positive and uniform_on_condition(f, lower, upper, delta0, eps0)
        }
        let delta0: Real satisfy {
            delta0.is_positive and uniform_on_condition(f, lower, upper, delta0, eps0)
        }
        delta0.is_positive
        not delta0.is_positive or not uniform_on_condition(f, lower, upper, delta0, eps0)
        not uniform_on_condition(f, lower, upper, delta0, eps0)
        uniform_on_condition(f, lower, upper, delta0, eps0)
        false
    }
    uniform_on(f, lower, upper)
}

/// The identity function is uniformly continuous on every closed interval.
theorem identity_function_is_uniform_on(lower: Real, upper: Real) {
    uniform_on(identity_fn[Real], lower, upper)
} by {
    identity_function_is_uniform
    uniform_imp_uniform_on(identity_fn[Real], lower, upper)
    uniform_on(identity_fn[Real], lower, upper)
}

/// The reciprocal function is uniformly continuous on [1, 2].
theorem reciprocal_real_uniform_on_1_2 {
    uniform_on(reciprocal_real, Real.1, Real.1 + Real.1)
} by {
    forall(eps: Real) {
        if eps.is_positive {
            forall(x: Real, y: Real) {
                if closed_interval_set(Real.1, Real.1 + Real.1).contains(x) and
                   closed_interval_set(Real.1, Real.1 + Real.1).contains(y) and
                   x.is_close(y, eps) {
                    closed_interval_set_lower_le(Real.1, Real.1 + Real.1, x)
                    Real.1 <= x
                    closed_interval_set_lower_le(Real.1, Real.1 + Real.1, y)
                    Real.1 <= y

                    // 0 < x and 0 < y, so x and y are nonzero.
                    Real.1.is_positive
                    pos_gt_zero(Real.1)
                    Real.1 > Real.0
                    Real.0 < Real.1
                    lt_of_lt_of_lte[Real](Real.0, Real.1, x)
                    Real.0 < x
                    lt_of_lt_of_lte[Real](Real.0, Real.1, y)
                    Real.0 < y
                    x != Real.0
                    y != Real.0
                    mul_not_zero[Real](x, y)
                    x * y != Real.0
                    gt_zero_imp_pos(x)
                    x.is_positive
                    gt_zero_imp_pos(y)
                    y.is_positive
                    mul_pos_pos(x, y)
                    (x * y).is_positive
                    pos_gt_zero(x * y)
                    Real.0 < x * y

                    // The difference of reciprocals is the reversed difference divided by the product.
                    reciprocal_real(x) = x.inverse
                    reciprocal_real(y) = y.inverse
                    inverse_sub_inverse(x, y)
                    x.inverse - y.inverse = (y - x) / (x * y)
                    (reciprocal_real(x) - reciprocal_real(y)).abs = ((y - x) / (x * y)).abs

                    abs_div(y - x, x * y)
                    ((y - x) / (x * y)).abs = (y - x).abs / (x * y).abs
                    pos_imp_eq_abs(x * y)
                    x * y = (x * y).abs
                    (y - x).abs / (x * y).abs = (y - x).abs / (x * y)
                    ((y - x) / (x * y)).abs = (y - x).abs / (x * y)
                    (reciprocal_real(x) - reciprocal_real(y)).abs = (y - x).abs / (x * y)

                    // x * y >= 1 because both lie in [1, 2].
                    lt_imp_lte[Real](Real.0, Real.1)
                    Real.0 <= Real.1
                    lte_trans[Real](Real.0, Real.1, y)
                    Real.0 <= y
                    mul_le_mul_of_nonneg_right[Real](Real.1, x, y)
                    Real.1 * y <= x * y
                    Real.1 * y = y
                    y <= x * y
                    lte_trans[Real](Real.1, y, x * y)
                    Real.1 <= x * y

                    // |y - x| / (x * y) <= |y - x| since x * y >= 1.
                    abs_gte_zero(y - x)
                    (y - x).abs >= Real.0
                    Real.0 <= (y - x).abs
                    mul_le_mul_of_nonneg_right[Real](Real.1, x * y, (y - x).abs)
                    (y - x).abs * Real.1 <= (y - x).abs * (x * y)
                    (y - x).abs * Real.1 = (y - x).abs
                    (y - x).abs <= (y - x).abs * (x * y)
                    div_le_div_pos((y - x).abs, (y - x).abs * (x * y), x * y)
                    (y - x).abs / (x * y) <= ((y - x).abs * (x * y)) / (x * y)
                    div_mul_cancel_left(x * y, (y - x).abs)
                    ((x * y) * (y - x).abs) / (x * y) = (y - x).abs
                    (y - x).abs * (x * y) = (x * y) * (y - x).abs
                    ((y - x).abs * (x * y)) / (x * y) = (y - x).abs
                    (y - x).abs / (x * y) <= (y - x).abs

                    (reciprocal_real(x) - reciprocal_real(y)).abs <= (y - x).abs
                    abs_neg(x - y)
                    (-(x - y)).abs = (x - y).abs
                    -(x - y) = y - x
                    (y - x).abs = (x - y).abs
                    (reciprocal_real(x) - reciprocal_real(y)).abs <= (x - y).abs
                    (x - y).abs < eps
                    lte_lt_trans((reciprocal_real(x) - reciprocal_real(y)).abs, (x - y).abs, eps)
                    (reciprocal_real(x) - reciprocal_real(y)).abs < eps
                    reciprocal_real(x).is_close(reciprocal_real(y), eps)
                }
            }
            uniform_on_condition(reciprocal_real, Real.1, Real.1 + Real.1, eps, eps)
            eps.is_positive and uniform_on_condition(reciprocal_real, Real.1, Real.1 + Real.1, eps, eps)
            exists(delta: Real) {
                delta.is_positive and uniform_on_condition(reciprocal_real, Real.1, Real.1 + Real.1, delta, eps)
            }
        }
    }
    if not uniform_on(reciprocal_real, Real.1, Real.1 + Real.1) {
        uniform_on(reciprocal_real, Real.1, Real.1 + Real.1) = forall(eps2: Real) {
            eps2.is_positive implies exists(delta2: Real) {
                delta2.is_positive and uniform_on_condition(reciprocal_real, Real.1, Real.1 + Real.1, delta2, eps2)
            }
        }
        not uniform_on(reciprocal_real, Real.1, Real.1 + Real.1)
        uniform_on(reciprocal_real, Real.1, Real.1 + Real.1) = forall(eps2: Real) {
            eps2.is_positive implies exists(delta2: Real) {
                delta2.is_positive and uniform_on_condition(reciprocal_real, Real.1, Real.1 + Real.1, delta2, eps2)
            }
        } and not uniform_on(reciprocal_real, Real.1, Real.1 + Real.1)
        eq_neg_transport(uniform_on(reciprocal_real, Real.1, Real.1 + Real.1), forall(eps2: Real) {
            eps2.is_positive implies exists(delta2: Real) {
                delta2.is_positive and uniform_on_condition(reciprocal_real, Real.1, Real.1 + Real.1, delta2, eps2)
            }
        })
        not forall(eps2: Real) {
            eps2.is_positive implies exists(delta2: Real) {
                delta2.is_positive and uniform_on_condition(reciprocal_real, Real.1, Real.1 + Real.1, delta2, eps2)
            }
        }
        exists(eps0: Real) {
            eps0.is_positive and forall(delta0: Real) {
                not delta0.is_positive or not uniform_on_condition(reciprocal_real, Real.1, Real.1 + Real.1, delta0, eps0)
            }
        }
        let eps0: Real satisfy {
            eps0.is_positive and forall(delta0: Real) {
                not delta0.is_positive or not uniform_on_condition(reciprocal_real, Real.1, Real.1 + Real.1, delta0, eps0)
            }
        }
        eps0.is_positive implies exists(delta0: Real) {
            delta0.is_positive and uniform_on_condition(reciprocal_real, Real.1, Real.1 + Real.1, delta0, eps0)
        }
        exists(delta0: Real) {
            delta0.is_positive and uniform_on_condition(reciprocal_real, Real.1, Real.1 + Real.1, delta0, eps0)
        }
        let delta0: Real satisfy {
            delta0.is_positive and uniform_on_condition(reciprocal_real, Real.1, Real.1 + Real.1, delta0, eps0)
        }
        delta0.is_positive
        not delta0.is_positive or not uniform_on_condition(reciprocal_real, Real.1, Real.1 + Real.1, delta0, eps0)
        not uniform_on_condition(reciprocal_real, Real.1, Real.1 + Real.1, delta0, eps0)
        uniform_on_condition(reciprocal_real, Real.1, Real.1 + Real.1, delta0, eps0)
        false
    }
    uniform_on(reciprocal_real, Real.1, Real.1 + Real.1)
}

// Heine–Cantor: a function continuous on a closed bounded interval is
// uniformly continuous there.  The statement below is left commented out:
// every classical proof needs one of two results that are not yet in the
// library, and building either from scratch is a project of its own.
//
//   - The sequence proof: assume f is continuous on [a, b] but not uniformly
//     continuous.  Then for some eps > 0 there are pairs x_n, y_n in [a, b]
//     with |x_n - y_n| < 1/(n+1) and |f(x_n) - f(y_n)| >= eps.  A bounded
//     sequence has a convergent subsequence (Bolzano–Weierstrass), so pass to
//     x_{n_k} -> p in [a, b]; then y_{n_k} -> p too, and continuity at p gives
//     |f(x_{n_k}) - f(y_{n_k})| -> 0, contradicting the lower bound eps.
//     Bolzano–Weierstrass is not proved in this library
//     (src/theorems1000/theorem_bolzano_weierstrass.ac states it but the
//     monotone-subsequence construction is commented out).  The cluster-set
//     machinery in src/real/ (sequence_cluster_set, sequence_cluster_sets.ac)
//     proves the cluster set is compact and closed but never that it is
//     nonempty without a convergence assumption, which is exactly the missing
//     Bolzano–Weierstrass content.
//
//   - The nested-interval proof: if [a, b] is bad at eps (arbitrarily close
//     pairs with |f| gap >= eps), then one of the two halves [a, m], [m, b]
//     of the midpoint m = (a + b)/2 is again bad.  Iterating yields nested
//     intervals [a_n, b_n] of length (b - a)/2^n, each bad; by Cantor's
//     intersection theorem (theorems1000_cantor_intersection, which IS
//     proved) they share a point p.  Continuity at p with tolerance eps/2
//     gives a neighbourhood of p inside which |f(x) - f(y)| < eps, but for
//     large n the whole interval [a_n, b_n] lies in that neighbourhood while
//     remaining bad — a contradiction.  The missing pieces are the recursive
//     halving construction (a classical choice of the bad half at each step,
//     expressible with `choose_of_exists` from data/basic/witness.ac) and the
//     dyadic length arithmetic; both are routine but lengthy.
//
//   - The open-cover proof: continuous-on-compact via the Lebesgue number.
//     This needs open-cover compactness of [a, b] (`is_compact[Real]`), which
//     is exactly the unproved Heine–Borel theorem
//     (src/theorems1000/theorem_heine_borel.ac is commented out); the library
//     only has the closed-and-bounded form `is_compact_real_set`.
//
// The definition of uniform continuity on [a, b], uniform-on implies
// continuity on [a, b], and the identity/reciprocal examples are proved
// above.

// theorem continuous_on_closed_imp_uniform_on(f: Real -> Real, lower: Real, upper: Real) {
//     continuous_on(f, lower, upper) and lower <= upper
//     implies
//     uniform_on(f, lower, upper)
// }
