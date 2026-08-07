from data.basic.functions import function_extensionality
from nat import Nat, alt_induction
from real.continuity_composition import constant_function_is_continuous
from real.continuity_pointwise_mul import continuous_pointwise_mul
from real.continuity_base import Real, continuous
from real.exp import pow_suc
from data.basic.function_algebra import pointwise_mul

/// The function obtained by raising f(x) to the fixed natural-number power n.
define pointwise_pow(f: Real -> Real, n: Nat, x: Real) -> Real {
    f(x).pow(n)
}

/// Raising to the zeroth power agrees with the constant function one.
theorem pointwise_pow_zero(f: Real -> Real) {
    pointwise_pow(f, Nat.0) = constant[Real, Real](Real.1)
} by {
    forall(x: Real) {
        pointwise_pow(f, Nat.0, x) = f(x).pow(Nat.0)
        f(x).pow(Nat.0) = Real.1
        constant[Real, Real](Real.1, x) = Real.1
        pointwise_pow(f, Nat.0, x) = constant[Real, Real](Real.1, x)
    }
    function_extensionality(pointwise_pow(f, Nat.0), constant[Real, Real](Real.1))
}

/// Raising to the successor power agrees with f times the lower power.
theorem pointwise_pow_suc(f: Real -> Real, n: Nat) {
    pointwise_pow(f, n.suc) = pointwise_mul[Real, Real](f, pointwise_pow(f, n))
} by {
    forall(x: Real) {
        pointwise_pow(f, n.suc, x) = f(x).pow(n.suc)
        pow_suc(f(x), n)
        f(x).pow(n.suc) = f(x) * f(x).pow(n)
        pointwise_pow(f, n, x) = f(x).pow(n)
        pointwise_mul[Real, Real](f, pointwise_pow(f, n), x) = f(x) * pointwise_pow(f, n, x)
        pointwise_mul[Real, Real](f, pointwise_pow(f, n), x) = f(x) * f(x).pow(n)
        pointwise_pow(f, n.suc, x) = pointwise_mul[Real, Real](f, pointwise_pow(f, n), x)
    }
    function_extensionality(pointwise_pow(f, n.suc), pointwise_mul[Real, Real](f, pointwise_pow(f, n)))
}

/// A natural-number power of a continuous real function is continuous.
theorem continuous_pointwise_pow(f: Real -> Real, n: Nat) {
    continuous(f) implies continuous(pointwise_pow(f, n))
} by {
    define p(x: Nat) -> Bool { continuous(f) implies continuous(pointwise_pow(f, x)) }

    // Base case: the zeroth power is the constant function one.
    pointwise_pow_zero(f)
    constant_function_is_continuous(Real.1)
    continuous(constant[Real, Real](Real.1))
    continuous(pointwise_pow(f, Nat.0))
    p(Nat.0)

    // Inductive step.
    forall(x: Nat) {
        if p(x) {
            if continuous(f) {
                continuous(pointwise_pow(f, x))
                continuous_pointwise_mul(f, pointwise_pow(f, x))
                continuous(pointwise_mul[Real, Real](f, pointwise_pow(f, x)))
                pointwise_pow_suc(f, x)
                continuous(pointwise_pow(f, x.suc))
            }
            p(x.suc)
        }
    }

    p(Nat.0) and forall(x: Nat) { p(x) implies p(x.suc) }
    alt_induction(p)
    forall(x: Nat) { p(x) }
    p(n)
}
