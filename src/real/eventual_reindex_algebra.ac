/// Algebraic consumers for reindexed bounded and vanishing real sequences.
from nat import Nat
from data.basic.functions import compose
from real.real_base import Real
from real.real_seq import add_seq
from real.real_series import mul_seq
from real.prod_seq import prod_seq
from real.bounded_seq import is_bounded_seq
from real.limits import vanishes, tends_to_infinity, is_subsequence_index, subsequence
from real.asymptotic_bounds import bounded_add_seq, bounded_mul_seq, bounded_prod_seq,
    bounded_mul_vanishing_seq, vanishing_mul_bounded_seq, vanishes_add_seq,
    vanishes_mul_seq
from real.eventual_bound_algebra import bounded_compose_tends_to_infinity,
    bounded_shift_add, bounded_subsequence, vanishes_compose_tends_to_infinity,
    vanishes_shift_add, vanishes_subsequence

/// The sum of bounded sequences remains bounded after reindexing along a map tending to infinity.
theorem bounded_add_seq_compose_tends_to_infinity(a: Nat -> Real, b: Nat -> Real, f: Nat -> Nat) {
    is_bounded_seq(a) and is_bounded_seq(b) and tends_to_infinity(f)
    implies is_bounded_seq(add_seq(compose(a, f), compose(b, f)))
} by {
    if is_bounded_seq(a) and is_bounded_seq(b) and tends_to_infinity(f) {
        bounded_compose_tends_to_infinity(a, f)
        is_bounded_seq(compose(a, f))
        bounded_compose_tends_to_infinity(b, f)
        is_bounded_seq(compose(b, f))
        bounded_add_seq(compose(a, f), compose(b, f))
        is_bounded_seq(add_seq(compose(a, f), compose(b, f)))
    }
}

/// The sum of vanishing sequences vanishes after reindexing along a map tending to infinity.
theorem vanishes_add_seq_compose_tends_to_infinity(a: Nat -> Real, b: Nat -> Real, f: Nat -> Nat) {
    vanishes(a) and vanishes(b) and tends_to_infinity(f)
    implies vanishes(add_seq(compose(a, f), compose(b, f)))
} by {
    if vanishes(a) and vanishes(b) and tends_to_infinity(f) {
        vanishes_compose_tends_to_infinity(a, f)
        vanishes(compose(a, f))
        vanishes_compose_tends_to_infinity(b, f)
        vanishes(compose(b, f))
        vanishes_add_seq(compose(a, f), compose(b, f))
        vanishes(add_seq(compose(a, f), compose(b, f)))
    }
}

/// A scalar multiple of a bounded sequence remains bounded after reindexing along a map tending to infinity.
theorem bounded_mul_seq_compose_tends_to_infinity(c: Real, a: Nat -> Real, f: Nat -> Nat) {
    is_bounded_seq(a) and tends_to_infinity(f)
    implies is_bounded_seq(mul_seq(c, compose(a, f)))
} by {
    if is_bounded_seq(a) and tends_to_infinity(f) {
        bounded_compose_tends_to_infinity(a, f)
        is_bounded_seq(compose(a, f))
        bounded_mul_seq(c, compose(a, f))
        is_bounded_seq(mul_seq(c, compose(a, f)))
    }
}

/// A scalar multiple of a vanishing sequence vanishes after reindexing along a map tending to infinity.
theorem vanishes_mul_seq_compose_tends_to_infinity(c: Real, a: Nat -> Real, f: Nat -> Nat) {
    vanishes(a) and tends_to_infinity(f)
    implies vanishes(mul_seq(c, compose(a, f)))
} by {
    if vanishes(a) and tends_to_infinity(f) {
        vanishes_compose_tends_to_infinity(a, f)
        vanishes(compose(a, f))
        vanishes_mul_seq(c, compose(a, f))
        vanishes(mul_seq(c, compose(a, f)))
    }
}

/// The product of bounded sequences remains bounded after reindexing along a map tending to infinity.
theorem bounded_prod_seq_compose_tends_to_infinity(a: Nat -> Real, b: Nat -> Real, f: Nat -> Nat) {
    is_bounded_seq(a) and is_bounded_seq(b) and tends_to_infinity(f)
    implies is_bounded_seq(prod_seq(compose(a, f), compose(b, f)))
} by {
    if is_bounded_seq(a) and is_bounded_seq(b) and tends_to_infinity(f) {
        bounded_compose_tends_to_infinity(a, f)
        is_bounded_seq(compose(a, f))
        bounded_compose_tends_to_infinity(b, f)
        is_bounded_seq(compose(b, f))
        bounded_prod_seq(compose(a, f), compose(b, f))
        is_bounded_seq(prod_seq(compose(a, f), compose(b, f)))
    }
}

/// A bounded factor times a vanishing factor vanishes after reindexing along a map tending to infinity.
theorem bounded_mul_vanishing_seq_compose_tends_to_infinity(
    a: Nat -> Real,
    b: Nat -> Real,
    f: Nat -> Nat
) {
    is_bounded_seq(a) and vanishes(b) and tends_to_infinity(f)
    implies vanishes(prod_seq(compose(a, f), compose(b, f)))
} by {
    if is_bounded_seq(a) and vanishes(b) and tends_to_infinity(f) {
        bounded_compose_tends_to_infinity(a, f)
        is_bounded_seq(compose(a, f))
        vanishes_compose_tends_to_infinity(b, f)
        vanishes(compose(b, f))
        bounded_mul_vanishing_seq(compose(a, f), compose(b, f))
        vanishes(prod_seq(compose(a, f), compose(b, f)))
    }
}

/// A vanishing factor times a bounded factor vanishes after reindexing along a map tending to infinity.
theorem vanishing_mul_bounded_seq_compose_tends_to_infinity(
    a: Nat -> Real,
    b: Nat -> Real,
    f: Nat -> Nat
) {
    vanishes(a) and is_bounded_seq(b) and tends_to_infinity(f)
    implies vanishes(prod_seq(compose(a, f), compose(b, f)))
} by {
    if vanishes(a) and is_bounded_seq(b) and tends_to_infinity(f) {
        vanishes_compose_tends_to_infinity(a, f)
        vanishes(compose(a, f))
        bounded_compose_tends_to_infinity(b, f)
        is_bounded_seq(compose(b, f))
        vanishing_mul_bounded_seq(compose(a, f), compose(b, f))
        vanishes(prod_seq(compose(a, f), compose(b, f)))
    }
}

/// The sum of bounded sequences remains bounded after deleting a finite prefix.
theorem bounded_add_seq_shift_add(a: Nat -> Real, b: Nat -> Real, k: Nat) {
    is_bounded_seq(a) and is_bounded_seq(b)
    implies is_bounded_seq(add_seq(compose(a, k.add), compose(b, k.add)))
} by {
    if is_bounded_seq(a) and is_bounded_seq(b) {
        bounded_shift_add(a, k)
        is_bounded_seq(compose(a, k.add))
        bounded_shift_add(b, k)
        is_bounded_seq(compose(b, k.add))
        bounded_add_seq(compose(a, k.add), compose(b, k.add))
        is_bounded_seq(add_seq(compose(a, k.add), compose(b, k.add)))
    }
}

/// The sum of vanishing sequences vanishes after deleting a finite prefix.
theorem vanishes_add_seq_shift_add(a: Nat -> Real, b: Nat -> Real, k: Nat) {
    vanishes(a) and vanishes(b)
    implies vanishes(add_seq(compose(a, k.add), compose(b, k.add)))
} by {
    if vanishes(a) and vanishes(b) {
        vanishes_shift_add(a, k)
        vanishes(compose(a, k.add))
        vanishes_shift_add(b, k)
        vanishes(compose(b, k.add))
        vanishes_add_seq(compose(a, k.add), compose(b, k.add))
        vanishes(add_seq(compose(a, k.add), compose(b, k.add)))
    }
}

/// The product of bounded sequences remains bounded after deleting a finite prefix.
theorem bounded_prod_seq_shift_add(a: Nat -> Real, b: Nat -> Real, k: Nat) {
    is_bounded_seq(a) and is_bounded_seq(b)
    implies is_bounded_seq(prod_seq(compose(a, k.add), compose(b, k.add)))
} by {
    if is_bounded_seq(a) and is_bounded_seq(b) {
        bounded_shift_add(a, k)
        is_bounded_seq(compose(a, k.add))
        bounded_shift_add(b, k)
        is_bounded_seq(compose(b, k.add))
        bounded_prod_seq(compose(a, k.add), compose(b, k.add))
        is_bounded_seq(prod_seq(compose(a, k.add), compose(b, k.add)))
    }
}

/// A bounded factor times a vanishing factor vanishes after deleting a finite prefix.
theorem bounded_mul_vanishing_seq_shift_add(a: Nat -> Real, b: Nat -> Real, k: Nat) {
    is_bounded_seq(a) and vanishes(b)
    implies vanishes(prod_seq(compose(a, k.add), compose(b, k.add)))
} by {
    if is_bounded_seq(a) and vanishes(b) {
        bounded_shift_add(a, k)
        is_bounded_seq(compose(a, k.add))
        vanishes_shift_add(b, k)
        vanishes(compose(b, k.add))
        bounded_mul_vanishing_seq(compose(a, k.add), compose(b, k.add))
        vanishes(prod_seq(compose(a, k.add), compose(b, k.add)))
    }
}

/// The sum of bounded sequences remains bounded along a selected subsequence.
theorem bounded_add_seq_subsequence(a: Nat -> Real, b: Nat -> Real, f: Nat -> Nat) {
    is_bounded_seq(a) and is_bounded_seq(b) and is_subsequence_index(f)
    implies is_bounded_seq(add_seq(subsequence(a, f), subsequence(b, f)))
} by {
    if is_bounded_seq(a) and is_bounded_seq(b) and is_subsequence_index(f) {
        bounded_subsequence(a, f)
        is_bounded_seq(subsequence(a, f))
        bounded_subsequence(b, f)
        is_bounded_seq(subsequence(b, f))
        bounded_add_seq(subsequence(a, f), subsequence(b, f))
        is_bounded_seq(add_seq(subsequence(a, f), subsequence(b, f)))
    }
}

/// The sum of vanishing sequences vanishes along a selected subsequence.
theorem vanishes_add_seq_subsequence(a: Nat -> Real, b: Nat -> Real, f: Nat -> Nat) {
    vanishes(a) and vanishes(b) and is_subsequence_index(f)
    implies vanishes(add_seq(subsequence(a, f), subsequence(b, f)))
} by {
    if vanishes(a) and vanishes(b) and is_subsequence_index(f) {
        vanishes_subsequence(a, f)
        vanishes(subsequence(a, f))
        vanishes_subsequence(b, f)
        vanishes(subsequence(b, f))
        vanishes_add_seq(subsequence(a, f), subsequence(b, f))
        vanishes(add_seq(subsequence(a, f), subsequence(b, f)))
    }
}

/// The product of bounded sequences remains bounded along a selected subsequence.
theorem bounded_prod_seq_subsequence(a: Nat -> Real, b: Nat -> Real, f: Nat -> Nat) {
    is_bounded_seq(a) and is_bounded_seq(b) and is_subsequence_index(f)
    implies is_bounded_seq(prod_seq(subsequence(a, f), subsequence(b, f)))
} by {
    if is_bounded_seq(a) and is_bounded_seq(b) and is_subsequence_index(f) {
        bounded_subsequence(a, f)
        is_bounded_seq(subsequence(a, f))
        bounded_subsequence(b, f)
        is_bounded_seq(subsequence(b, f))
        bounded_prod_seq(subsequence(a, f), subsequence(b, f))
        is_bounded_seq(prod_seq(subsequence(a, f), subsequence(b, f)))
    }
}

/// A bounded factor times a vanishing factor vanishes along a selected subsequence.
theorem bounded_mul_vanishing_seq_subsequence(a: Nat -> Real, b: Nat -> Real, f: Nat -> Nat) {
    is_bounded_seq(a) and vanishes(b) and is_subsequence_index(f)
    implies vanishes(prod_seq(subsequence(a, f), subsequence(b, f)))
} by {
    if is_bounded_seq(a) and vanishes(b) and is_subsequence_index(f) {
        bounded_subsequence(a, f)
        is_bounded_seq(subsequence(a, f))
        vanishes_subsequence(b, f)
        vanishes(subsequence(b, f))
        bounded_mul_vanishing_seq(subsequence(a, f), subsequence(b, f))
        vanishes(prod_seq(subsequence(a, f), subsequence(b, f)))
    }
}

/// A vanishing factor times a bounded factor vanishes along a selected subsequence.
theorem vanishing_mul_bounded_seq_subsequence(a: Nat -> Real, b: Nat -> Real, f: Nat -> Nat) {
    vanishes(a) and is_bounded_seq(b) and is_subsequence_index(f)
    implies vanishes(prod_seq(subsequence(a, f), subsequence(b, f)))
} by {
    if vanishes(a) and is_bounded_seq(b) and is_subsequence_index(f) {
        vanishes_subsequence(a, f)
        vanishes(subsequence(a, f))
        bounded_subsequence(b, f)
        is_bounded_seq(subsequence(b, f))
        vanishing_mul_bounded_seq(subsequence(a, f), subsequence(b, f))
        vanishes(prod_seq(subsequence(a, f), subsequence(b, f)))
    }
}
