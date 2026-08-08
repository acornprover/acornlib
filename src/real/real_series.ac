from data.basic.functions import compose
from nat import Nat, lte_trans, pow_zero
from rat import Rat
from list import partial, sum, map, List
from list import map_add, sum_add, partial_scalar_mul, partial_add, partial_pointwise_eq
from algebra.add_semigroup import add_fn
from algebra.semigroup import mul_fn
from real.real_set import Real
from real.real_seq import cauchy_bound, tail_bound, add_seq, neg_rat_seq, converges_imp_converges_to, eventual_ub, eventual_lb, lt_converges_to_imp_lb, gt_converges_to_imp_ub, eventual_eq, eq_converges, eq_imp_limit, converges_to_unique
from real.real_ring import converges, converges_to, limit, lift_seq, mul_nonneg, mul_abs
from real.real_base import lte_add_right, close_imp_bounds
from order import is_monotone, is_antitone, monotone_from_forall, antitone_from_forall

numerals Real

// This file defines infinite series and proves theorems about them.

attributes Real {
    // Placeholder to let other modules import Real from here.
}

// seq_lte is whether every element of the sequence is lte.
define seq_lte(a: Nat -> Real, b: Nat -> Real) -> Bool {
    forall(n: Nat) {
        a(n) <= b(n)
    }
}

/// Product function for double sums.
/// Creates the outer product of two sequences: prod_fn(a, b)(i, j) = a(i) * b(j).
define prod_fn(a: Nat -> Real, b: Nat -> Real) -> ((Nat, Nat) -> Real) {
    function(i: Nat, j: Nat) { a(i) * b(j) }
}

theorem partial_suc(a: Nat -> Real, n: Nat) {
    partial(a, n.suc) = partial(a, n) + a(n)
} by {
    // Simplify the left hand side.
}

// Theorem: If seq_lte(a, b), then their partials also obey seq_lte.
theorem partial_seq_lte(a: Nat -> Real, b: Nat -> Real) {
    seq_lte(a, b) implies seq_lte(partial(a), partial(b))
} by {
    // Define a predicate for our induction
    define p(n: Nat) -> Bool {
        partial(a)(n) <= partial(b)(n)
    }

    p(Nat.0)

    // Inductive step
    forall(n: Nat) {
        if p(n) {
            partial(a, n) + a(n) <= partial(b, n) + b(n)
            p(n.suc)
        }
    }

    // By induction, p holds for all n
}

// This definition of increasing is not strict.
// It includes less-than-or-equal.
define is_increasing(a: Nat -> Real) -> Bool {
    forall(n: Nat) {
        a(n) <= a(n.suc)
    }
}

theorem distant_increasing(a: Nat -> Real, m: Nat, n: Nat) {
    is_increasing(a) and m <= n implies a(m) <= a(n)
} by {
    if is_increasing(a) and m <= n {
        define p(k: Nat) -> Bool {
            forall(j: Nat) {
                j + k <= n implies a(j) <= a(j + k)
            }
        }
        forall(j: Nat) {
            if j + Nat.0 <= n {
                j + Nat.0 = j
                a(j) <= a(j)
            }
        }
        forall(j: Nat) {
            j + Nat.0 <= n implies a(j) <= a(j + Nat.0)
        }
        p(Nat.0)

        forall(k: Nat) {
            if p(k) {
                forall(j: Nat) {
                    if j + k.suc <= n {
                        j + k.suc = (j + k).suc
                        j + k <= n or n < j + k
                        not n < j + k or n < (j + k).suc
                        not j + k.suc <= n or not n < j + k.suc
                        j + k <= n
                        a(j) <= a(j + k)
                        a(j + k) <= a((j + k).suc)
                        (j + k).suc = j + k.suc
                        a(j) <= a(j + k.suc)
                    }
                }
                p(k.suc)
            }
        }

        let k: Nat satisfy {
            m + k = n
        }
        p(k)
        a(m) <= a(m + k)
        a(m + k) = a(n)
        a(m) <= a(n)
    }
}

/// A monotone map from natural numbers to reals is an increasing sequence.
theorem monotone_is_increasing(a: Nat -> Real) {
    is_monotone(a) implies is_increasing(a)
} by {
    if is_monotone(a) {
        forall(n: Nat) {
            n <= n.suc
            a(n) <= a(n.suc)
        }
    }
}

/// An increasing sequence is a monotone map from the naturals.
theorem increasing_is_monotone(a: Nat -> Real) {
    is_increasing(a) implies is_monotone(a)
} by {
    if is_increasing(a) {
        forall(m: Nat, n: Nat) {
            if m <= n {
                distant_increasing(a, m, n)
                a(m) <= a(n)
            }
        }
        monotone_from_forall(a)
        is_monotone(a)
    }
}

/// A local increasing sequence and a monotone map from the naturals are the same condition.
theorem increasing_iff_monotone(a: Nat -> Real) {
    is_increasing(a) = is_monotone(a)
} by {
    if is_increasing(a) {
        increasing_is_monotone(a)
        is_monotone(a)
    }
    if is_monotone(a) {
        monotone_is_increasing(a)
        is_increasing(a)
    }
    is_increasing(a) = is_monotone(a)
}

/// A monotone real sequence preserves order between arbitrary indices.
theorem distant_monotone(a: Nat -> Real, m: Nat, n: Nat) {
    is_monotone(a) and m <= n implies a(m) <= a(n)
} by {
    if is_monotone(a) and m <= n {
        is_increasing(a)
        distant_increasing(a, m, n)
        a(m) <= a(n)
    }
}

define is_upper_bound(a: Nat -> Real, b: Real) -> Bool {
    forall(n: Nat) {
        a(n) <= b
    }
}

/// A sequence has an upper bound if some real number bounds every term from above.
define has_upper_bound_seq(a: Nat -> Real) -> Bool {
    exists(ub: Real) {
        is_upper_bound(a, ub)
    }
}

define is_least_upper_bound(a: Nat -> Real, l: Real) -> Bool {
    is_upper_bound(a, l) and forall(x: Real) {
        x < l implies not is_upper_bound(a, x)
    }
}

define is_lower_bound(a: Nat -> Real, b: Real) -> Bool {
    forall(n: Nat) {
        b <= a(n)
    }
}

/// A sequence has a lower bound if some real number bounds every term from below.
define has_lower_bound_seq(a: Nat -> Real) -> Bool {
    exists(lb: Real) {
        is_lower_bound(a, lb)
    }
}

// Theorem: If a sequence is increasing and converges, then its limit is an upper bound
theorem increasing_convergent_bounded_by_limit(a: Nat -> Real) {
    is_increasing(a) and converges(a)
    implies is_upper_bound(a, limit(a))
} by {
    // We'll prove that for any n, a(n) <= limit(a)
    forall(n: Nat) {
        // Suppose for contradiction that a(n) > limit(a)
        if a(n) > limit(a) {
            limit(a) < a(n)

            // Find a value that's between limit(a) and a(n)
            let ub: Real satisfy {
                limit(a) < ub and ub < a(n)
            }

            // Since the sequence converges to limit(a), there must be a point
            // after which all sequence values are < ub
            let eps = ub - limit(a)
            eps.is_positive

            // By the definition of convergence, find a point where the sequence
            // is within eps of the limit for all indices beyond that point
            converges_to(a, limit(a))
            exists(big_n: Nat) {
                tail_bound(a, limit(a), big_n, eps)
            }
            let (big_n: Nat) satisfy {
                tail_bound(a, limit(a), big_n, eps)
            }

            // Since a(n) > ub, we know n can't be >= big_n (or a(n) would be within eps of limit(a))
            // So either n < big_n or we have a contradiction
            // Case 1: n < big_n
            if n < big_n {
                // Since sequence is increasing, a(n) <= a(big_n)

                // But since a(big_n) is within eps of limit(a), we have a(big_n) < ub
                // Being close means the absolute difference is < eps
                // For real numbers, if they're close and one is less than another plus epsilon
                // then the first is less than the second plus epsilon
                // And since eps = ub - limit(a), we have limit(a) + eps = ub
                big_n <= big_n
                a(big_n).is_close(limit(a), eps)
                a(big_n) < limit(a) + eps
                limit(a) + eps = ub
                a(big_n) < ub

                // So a(n) <= a(big_n) < ub, contradicting our assumption that a(n) > ub
                n <= big_n
                a(n) <= a(big_n)
                ub < a(big_n)
                ub < ub
                false
            } else {
                // Case 2: n >= big_n
                // Then a(n) is within eps of limit(a)
                // Being close means the absolute difference is < eps
                // For real numbers, if they're close and one is less than another plus epsilon
                // then the first is less than the second plus epsilon
                // And since eps = ub - limit(a), we have limit(a) + eps = ub
                n < big_n or big_n <= n
                big_n <= n
                a(n).is_close(limit(a), eps)
                a(n) < limit(a) + eps
                limit(a) + eps = ub
                a(n) < ub

                // Contradicting our assumption that a(n) > ub
                false
            }

            // Either way, we get a contradiction, so a(n) <= limit(a)
        }

        // Therefore, a(n) <= limit(a)
    }
}

define image(a: Nat -> Real, x: Real) -> Bool {
    exists(n: Nat) {
        a(n) = x
    }
}

theorem ub_imp_image_ub(a: Nat -> Real, b: Real) {
    is_upper_bound(a, b)
    implies b.is_set_upper_bound(image(a))
} by {
    if is_upper_bound(a, b) {
        forall(x: Real) {
            if image(a, x) {
                let n: Nat satisfy {
                    a(n) = x
                }
                a(n) <= b
                x <= b
            }
        }
    }
}

// If there's an upper bound, there's a least upper bound.
theorem ub_imp_lub(a: Nat -> Real, b: Real) {
    is_upper_bound(a, b) implies exists(c: Real) {
        is_least_upper_bound(a, c)
    }
} by {
    let c: Real satisfy {
        c.is_set_least_upper_bound(image(a))
    }
    c.is_set_upper_bound(image(a))
    forall(n: Nat) {
        image(a, a(n))
        a(n) <= c
    }
    is_upper_bound(a, c)
    forall(x: Real) {
        if x < c {
            if is_upper_bound(a, x) {
                x.is_set_upper_bound(image(a))
                c <= x
                x < c = c > x
                not c <= x or not c > x
            }
            not is_upper_bound(a, x)
        }
    }
    is_least_upper_bound(a, c)
}

// The monotone convergence principle states that any increasing sequence
// that is bounded above converges
theorem monotone_convergence_principle(a: Nat -> Real, b: Real) {
    is_increasing(a) and is_upper_bound(a, b) implies converges(a)
} by {
    let l: Real satisfy {
        is_least_upper_bound(a, l)
    }

    forall(eps: Real) {
        if eps.is_positive {
            // Find epsilon/2 for triangle inequality
            let eps2: Real satisfy {
                eps2.is_positive and eps2 + eps2 < eps
            }

            // Since l is the least upper bound, l-eps2 is not an upper bound
            let x = l - eps2
            x < l

            // If x is not an upper bound, there exists some n such that a(n) > x
            let n: Nat satisfy {
                not a(n) <= x
            }
            // We know a(n) > x = l - eps2

            // For all indices i,j ≥ n, prove they're within epsilon of each other
            forall(i: Nat, j: Nat) {
                if n <= i and n <= j {
                    a(i) <= l
                    a(i) < l + eps2
                    // Show l - eps2 < a(i) via increasing sequence
                    a(n) <= a(i)
                    a(i) <= x or a(i) > x
                    x < a(i) = a(i) > x
                    l - eps2 < a(i)
                    a(i).is_close(l, eps2)
                    (a(i) - l).abs < eps2 = a(i).is_close(l, eps2)
                    (a(i) - l).abs < eps2
                    a(j) <= l
                    a(j) < l + eps2
                    // Show l - eps2 < a(j) via increasing sequence
                    a(n) <= a(j)
                    a(j) <= x or a(j) > x
                    x < a(j) = a(j) > x
                    l - eps2 < a(j)
                    a(j).is_close(l, eps2)
                    (a(j) - l).abs < eps2 = a(j).is_close(l, eps2)
                    (a(j) - l).abs < eps2
                    (a(i) - a(j)).abs < eps2 + eps2
                    a(i).is_close(a(j), eps)
                }
            }

            // We've proven that for all i,j ≥ n,
            // a(i) and a(j) are within epsilon of each other.
            // This is exactly the definition of cauchy_condition
            exists(k0: Nat, k1: Nat) {
                n <= k0 and n <= k1 and not a(k0).is_close(a(k1), eps)
            } or cauchy_bound(a, n, eps)
            cauchy_bound(a, n, eps)
        }
    }
}

/// An increasing sequence with an upper bound converges.
theorem increasing_bounded_above_converges(a: Nat -> Real) {
    is_increasing(a) and has_upper_bound_seq(a) implies converges(a)
} by {
    if is_increasing(a) and has_upper_bound_seq(a) {
        let ub: Real satisfy {
            is_upper_bound(a, ub)
        }
        converges(a)
    }
}

/// A monotone real sequence with an upper bound converges.
theorem monotone_bounded_above_converges(a: Nat -> Real, b: Real) {
    is_monotone(a) and is_upper_bound(a, b) implies converges(a)
} by {
    if is_monotone(a) and is_upper_bound(a, b) {
        is_increasing(a)
        converges(a)
    }
}

/// A monotone real sequence with some upper bound converges.
theorem monotone_has_upper_bound_converges(a: Nat -> Real) {
    is_monotone(a) and has_upper_bound_seq(a) implies converges(a)
} by {
    if is_monotone(a) and has_upper_bound_seq(a) {
        is_increasing(a)
        converges(a)
    }
}

/// A convergent monotone real sequence is bounded above by its limit.
theorem monotone_convergent_bounded_by_limit(a: Nat -> Real) {
    is_monotone(a) and converges(a) implies is_upper_bound(a, limit(a))
} by {
    if is_monotone(a) and converges(a) {
        is_increasing(a)
        is_upper_bound(a, limit(a))
    }
}

theorem nonneg_partial_increasing(a: Nat -> Real) {
    is_lower_bound(a, Real.0) implies is_increasing(partial(a))
} by {
    forall(n: Nat) {
        if is_lower_bound(a, Real.0) {
            Real.0 <= a(n)
            partial(a, n) + Real.0 = partial(a, n)
            partial(a, n) + Real.0 <= partial(a, n) + a(n)
            partial(a, n) + a(n) = partial(a, n.suc)
            partial(a, n) <= partial(a, n.suc)
        }
    }
}

/// Partial sums are monotone as maps for nonnegative sequences.
theorem nonneg_partial_monotone(a: Nat -> Real) {
    is_lower_bound(a, Real.0) implies is_monotone(partial(a))
} by {
    if is_lower_bound(a, Real.0) {
        nonneg_partial_increasing(a)
        is_increasing(partial(a))
        increasing_is_monotone(partial(a))
        is_monotone(partial(a))
    }
}

/// Partial sums are monotone for nonnegative sequences.
theorem partial_monotone(a: Nat -> Real, n: Nat) {
    is_lower_bound(a, Real.0) implies partial(a, n) <= partial(a, n.suc)
}

theorem nonneg_partial_bounded_above(a: Nat -> Real) {
    is_lower_bound(a, Real.0) and converges(partial(a))
    implies is_upper_bound(partial(a), limit(partial(a)))
}

theorem seq_lte_ub(a: Nat -> Real, b: Nat -> Real, ub: Real) {
    seq_lte(a, b) and is_upper_bound(b, ub)
    implies is_upper_bound(a, ub)
} by {
    forall(n: Nat) {
        if seq_lte(a, b) and is_upper_bound(b, ub) {
            a(n) <= b(n)
            b(n) <= ub
            a(n) <= ub
        }
    }
}

/// If two sequences satisfy a <= b pointwise and both converge, their limits preserve the order.
theorem seq_lte_preserves_limit(a: Nat -> Real, b: Nat -> Real) {
    seq_lte(a, b) and converges(a) and converges(b)
    implies limit(a) <= limit(b)
} by {
    if seq_lte(a, b) and converges(a) and converges(b) {
        // Proof by contradiction
        if limit(a) > limit(b) {
            // Find a rational strictly between the two limits
            let r: Rat satisfy {
                limit(a) > Real.from_rat(r) and Real.from_rat(r) > limit(b)
            }

            let r_real = Real.from_rat(r)

            // Get convergence-to statements
            // Since limit(b) < r_real and b converges to limit(b),
            // eventually b(n) <= r_real (by gt_converges_to_imp_ub)
            r_real > limit(b)
            eventual_ub(b, r_real)

            let n1: Nat satisfy {
                forall(i: Nat) {
                    n1 <= i implies b(i) <= r_real
                }
            }

            // Since limit(a) > r_real and a converges to limit(a),
            // eventually a(n) >= r_real (by lt_converges_to_imp_lb)
            r_real < limit(a)
            eventual_lb(a, r_real)

            let n2: Nat satisfy {
                forall(i: Nat) {
                    n2 <= i implies a(i) >= r_real
                }
            }

            // Take any n >= max(n1, n2)
            let n = n1.max(n2)
            n1 <= n
            n2 <= n

            // Then a(n) >= r_real and b(n) <= r_real
            a(n) >= r_real
            b(n) <= r_real

            // We also know a(n) <= b(n) from seq_lte
            a(n) <= b(n)

            // Now use that limit(a) > r_real.
            // Since a converges to limit(a) and r_real < limit(a),
            // we can get a contradiction by showing a(n) is strictly > r_real
            // for large enough n.

            let eps_a = limit(a) - r_real
            eps_a.is_positive

            let eps_a_half: Real satisfy {
                eps_a_half.is_positive and eps_a_half + eps_a_half < eps_a
            }

            // Eventually a(n) is within eps_a_half of limit(a)
            converges_to(a, limit(a))
            exists(big_n3: Nat) {
                tail_bound(a, limit(a), big_n3, eps_a_half)
            }
            let big_n5: Nat satisfy {
                tail_bound(a, limit(a), big_n5, eps_a_half)
            }
            forall(i: Nat) {
                if big_n5 <= i {
                    a(i).is_close(limit(a), eps_a_half)
                }
            }
            exists(big_n6: Nat) {
                forall(i: Nat) {
                    big_n6 <= i implies a(i).is_close(limit(a), eps_a_half)
                }
            }
            exists(big_n4: Nat) {
                forall(i: Nat) {
                    big_n4 <= i implies a(i).is_close(limit(a), eps_a_half)
                }
            }
            let n3: Nat satisfy {
                forall(i: Nat) {
                    n3 <= i implies a(i).is_close(limit(a), eps_a_half)
                }
            }

            // For i >= max(n, n3):
            let nn = n.max(n3)
            n <= nn
            n3 <= nn

            a(nn).is_close(limit(a), eps_a_half)
            // From is_close, we get a(nn) > limit(a) - eps_a_half
            limit(a) - eps_a_half < a(nn)

            // Now: limit(a) = r_real + eps_a
            limit(a) = r_real + eps_a

            // So: limit(a) - eps_a_half < a(nn)
            limit(a) - eps_a_half < a(nn)

            // Since eps_a_half + eps_a_half < eps_a:
            eps_a_half + eps_a_half < eps_a

            // And eps_a = limit(a) - r_real:
            eps_a = limit(a) - r_real

            // We have: eps_a_half + eps_a_half < limit(a) - r_real
            eps_a_half + eps_a_half < limit(a) - r_real

            // This implies: r_real + eps_a_half < limit(a) - eps_a_half
            eps_a - (eps_a_half + eps_a_half) = eps_a - eps_a_half - eps_a_half
            limit(a) - (r_real + eps_a_half) = limit(a) - r_real - eps_a_half
            limit(a) - eps_a_half - (r_real + eps_a_half) = limit(a) - (r_real + eps_a_half) - eps_a_half
            (eps_a - (eps_a_half + eps_a_half)).is_positive
            (limit(a) - eps_a_half - (r_real + eps_a_half)).is_positive
            r_real + eps_a_half < limit(a) - eps_a_half

            // Since eps_a_half.is_positive: r_real < r_real + eps_a_half
            r_real < r_real + eps_a_half

            // By transitivity: r_real < limit(a) - eps_a_half
            r_real < limit(a) - eps_a_half
            limit(a) - eps_a_half < a(nn)
            r_real < a(nn)

            // But we also have a(nn) <= b(nn) <= r_real (since nn >= n)
            a(nn) <= b(nn)
            b(nn) <= r_real
            a(nn) <= r_real

            // So a(nn) <= r_real and r_real < a(nn), which is a contradiction
            false
        }

        // Therefore limit(a) <= limit(b)
    }
}

theorem comparison_test(a: Nat -> Real, b: Nat -> Real) {
    is_lower_bound(a, Real.0)
    and seq_lte(a, b)
    and converges(partial(b))
    implies converges(partial(a))
} by {
    if is_lower_bound(a, Real.0) and seq_lte(a, b) and converges(partial(b)) {
        // b is also nonnegative since a <= b and a >= 0
        forall(n: Nat) {
            Real.0 <= a(n)
            a(n) <= b(n)
            Real.0 <= b(n)
        }
        is_lower_bound(b, Real.0)
        // Partial sums of a are increasing
        is_increasing(partial(a))
        // Partial sums satisfy a <= b pointwise
        seq_lte(partial(a), partial(b))
        // b's partial sums are bounded by limit(partial(b))
        is_upper_bound(partial(b), limit(partial(b)))
        // So a's partial sums are also bounded
        is_upper_bound(partial(a), limit(partial(b)))
        // Increasing and bounded implies convergent
        converges(partial(a))
    }
}

define mul_seq(a: Real, b: Nat -> Real, n: Nat) -> Real {
    a * b(n)
}

theorem const_converges(a: Real) {
    converges(constant[Nat, Real](a))
} by {
    forall(n: Nat) {
        constant(a, n) = a
    }
    eventual_eq(constant[Nat, Real](a), a)
}

theorem const_limit(a: Real) {
    limit(constant[Nat, Real](a)) = a
} by {
    forall(n: Nat) {
        constant(a, n) = a
    }
    eventual_eq(constant[Nat, Real](a), a)
}

theorem const_converges_to(a: Real) {
    converges_to(constant[Nat, Real](a), a)
} by {
}

theorem mul_seq_zero(a: Nat -> Real) {
    mul_seq(Real.0, a) = constant[Nat, Real](Real.0)
} by {
    forall(n: Nat) {
        mul_seq(Real.0, a, n) = Real.0 * a(n)
        Real.0 * a(n) = Real.0
        constant(Real.0, n) = Real.0
        mul_seq(Real.0, a, n) = constant(Real.0, n)
    }
}

theorem mul_seq_converges_to(a: Real, b: Nat -> Real) {
    converges(b) implies converges_to(mul_seq(a, b), a * limit(b))
} by {
    forall(eps: Real) {
        if eps.is_positive {
            let eps2: Real satisfy {
                eps2.is_positive and a.abs * eps2 < eps
            }
            let n: Nat satisfy {
                tail_bound(b, limit(b), n, eps2)
            }
            forall(i: Nat) {
                if n <= i {
                    a.abs * (b(i) - limit(b)).abs <= a.abs * eps2
                    mul_seq(a, b, i) = a * b(i)
                    a * b(i) - a * limit(b) = a * (b(i) - limit(b))
                    a.abs * (b(i) - limit(b)).abs = (a * (b(i) - limit(b))).abs
                    (a * (b(i) - limit(b))).abs < eps
                    (mul_seq(a, b, i) - a * limit(b)).abs < eps
                    (mul_seq(a, b, i) - a * limit(b)).abs < eps = mul_seq(a, b, i).is_close(a * limit(b), eps)
                    mul_seq(a, b)(i).is_close(a * limit(b), eps)
                }
            }
            exists(k0: Nat) {
                n <= k0 and not mul_seq(a, b, k0).is_close(a * limit(b), eps)
            } or tail_bound(mul_seq(a, b), a * limit(b), n, eps)
            tail_bound(mul_seq(a, b), a * limit(b), n, eps)
            exists(n0: Nat) {
                tail_bound(mul_seq(a, b), a * limit(b), n0, eps)
            }
        }
    }
    forall(eps: Real) {
        eps.is_positive implies exists(n0: Nat) {
            tail_bound(mul_seq(a, b), a * limit(b), n0, eps)
        }
    }
    exists(k0: Real) {
        k0.is_positive and forall(x0: Nat) {
            not tail_bound(mul_seq(a, b), a * limit(b), x0, k0)
        }
    } or converges_to(mul_seq(a, b), a * limit(b))
    converges_to(mul_seq(a, b), a * limit(b))
}

theorem mul_seq_one(a: Nat -> Real) {
    mul_seq(Real.1, a) = a
} by {
    forall(n: Nat) {
        mul_seq(Real.1, a, n) = Real.1 * a(n)
        Real.1 * a(n) = a(n)
        mul_seq(Real.1, a, n) = a(n)
    }
}

theorem mul_seq_neg_one_converges_converse(a: Nat -> Real) {
    converges(mul_seq(-Real.1, a)) implies converges(a)
} by {
    converges(mul_seq(-Real.1, mul_seq(-Real.1, a)))
    forall(n: Nat) {
        mul_seq(-Real.1, mul_seq(-Real.1, a), n) = -Real.1 * mul_seq(-Real.1, a, n)
        mul_seq(-Real.1, a, n) = -Real.1 * a(n)
        mul_seq((-Real.1) * -Real.1, a, n) = ((-Real.1) * -Real.1) * a(n)
        -Real.1 * (-Real.1 * a(n)) = ((-Real.1) * -Real.1) * a(n)
        mul_seq(-Real.1, mul_seq(-Real.1, a), n) = mul_seq((-Real.1) * -Real.1, a, n)
    }
    (-Real.1) * -Real.1 = Real.1
    mul_seq((-Real.1) * -Real.1, a) = mul_seq(Real.1, a)
    forall(n: Nat) {
        mul_seq(Real.1, a, n) = Real.1 * a(n)
        Real.1 * a(n) = a(n)
        mul_seq(Real.1, a, n) = a(n)
    }
    mul_seq(Real.1, a) = a
}

theorem mul_seq_comm(a: Real, b: Real, c: Nat -> Real) {
    mul_seq(a, mul_seq(b, c)) = mul_seq(b, mul_seq(a, c))
} by {
    forall(n: Nat) {
        mul_seq(a, mul_seq(b, c), n) = a * mul_seq(b, c, n)
        mul_seq(b, c, n) = b * c(n)
        mul_seq(b, mul_seq(a, c), n) = b * mul_seq(a, c, n)
        mul_seq(a, c, n) = a * c(n)
        a * (b * c(n)) = a * b * c(n)
        b * (a * c(n)) = b * a * c(n)
        b * a = a * b
        mul_seq(a, mul_seq(b, c))(n) = mul_seq(b, mul_seq(a, c))(n)
    }
}

theorem mul_seq_combine(a: Real, b: Real, c: Nat -> Real) {
    mul_seq(a * b, c) = mul_seq(a, mul_seq(b, c))
} by {
    forall(n: Nat) {
        mul_seq(a * b, c, n) = (a * b) * c(n)
        mul_seq(a, mul_seq(b, c), n) = a * mul_seq(b, c, n)
        mul_seq(b, c, n) = b * c(n)
        (a * b) * c(n) = a * b * c(n)
        a * (b * c(n)) = a * b * c(n)
        mul_seq(a * b, c, n) = mul_seq(a, mul_seq(b, c), n)
    }
}

theorem converges_mul_seq(a: Real, b: Nat -> Real) {
    converges(b) implies converges(mul_seq(a, b))
}

// Maybe it would have been better to not define partial this way.
theorem partial_zero(a: Nat -> Real) {
    partial(a, Nat.0) = Real.0
}

/// If all terms before index m are zero, then the partial sum is zero.
theorem partial_all_zeros(g: Nat -> Real, m: Nat) {
    (forall(j: Nat) { j < m implies g(j) = Real.0 })
    implies
    partial(g, m) = Real.0
} by {
    define p(k: Nat) -> Bool {
        (forall(j: Nat) { j < k implies g(j) = Real.0 })
        implies
        partial(g, k) = Real.0
    }
    // Base case
    if forall(j: Nat) { j < Nat.0 implies g(j) = Real.0 } {
    }
    p(Nat.0)

    // Inductive step
    forall(k: Nat) {
        if p(k) {
            if forall(j: Nat) { j < k.suc implies g(j) = Real.0 } {
                // g(j) = 0 for all j < k
                forall(j: Nat) {
                    if j < k {
                        j < k.suc
                        g(j) = Real.0
                    }
                }
                k < k.suc
                g(k) = Real.0

                // By IH: partial(g, k) = 0
                partial(g, k) = Real.0

                // Therefore partial(g, k.suc) = 0 + 0 = 0
                partial(g, k) + g(k) = partial(g, k.suc)
                partial(g, k) + Real.0 = partial(g, k)
                partial(g, k.suc) = Real.0
                p(k.suc)
            }
            if forall(j: Nat) { j < k.suc implies g(j) = Real.0 } {
                forall(j: Nat) {
                    if j < k {
                        j < k.suc
                        g(j) = Real.0
                    }
                }
                partial(g, k) = Real.0
                k < k.suc
                g(k) = Real.0
                partial(g, k) + g(k) = partial(g, k.suc)
                partial(g, k) + Real.0 = partial(g, k)
                partial(g, k.suc) = Real.0
            }
            (forall(j: Nat) { j < k.suc implies g(j) = Real.0 }) implies partial(g, k.suc) = Real.0
            not (forall(j: Nat) { j < k.suc implies g(j) = Real.0 }) or partial(g, k.suc) = Real.0
            p(k.suc)
        }
    }

    p(m)
}

theorem partial_add_seq_comm(a: Nat -> Real, b: Nat -> Real) {
    partial(add_seq(a, b)) = add_seq(partial(a), partial(b))
} by {
    forall(n: Nat) {
        add_seq(a, b, n) = a(n) + b(n)
        add_fn(a, b, n) = a(n) + b(n)
        add_seq(a, b)(n) = add_fn(a, b)(n)
    }
    add_seq(a, b) = add_fn(a, b)
    forall(n: Nat) {
        partial(a, n) + partial(b, n) = partial(add_fn(a, b), n)
        add_seq(partial(a), partial(b), n) = partial(a, n) + partial(b, n)
        partial(add_seq(a, b), n) = partial(add_fn(a, b), n)
        partial(add_seq(a, b), n) = add_seq(partial(a), partial(b), n)
    }

    // Define a predicate for induction on n
    define p(n: Nat) -> Bool {
        partial(add_seq(a, b), n) = add_seq(partial(a), partial(b))(n)
    }

    // Base case: n = 0
    partial(add_seq(a, b), Nat.0) = Real.0
    partial(a, Nat.0) + partial(b, Nat.0) = add_seq(partial(a), partial(b), Nat.0)
    add_seq(partial(a), partial(b), Nat.0) = Real.0
    p(Nat.0)

    // Inductive step
    forall(n: Nat) {
        if p(n) {
            // By the induction hypothesis
            add_seq(partial(a), partial(b), n) = partial(add_seq(a, b), n)

            // Now we need to prove for n.suc
            a(n) + b(n) = add_seq(a, b, n)
            partial(a, n) + partial(b, n) = add_seq(partial(a), partial(b), n)
            partial(a, n.suc) + partial(b, n.suc) = add_seq(partial(a), partial(b), n.suc)
            partial(add_seq(a, b), n) + add_seq(a, b, n) = partial(add_seq(a, b), n.suc)

            // Therefore
            partial(add_seq(a, b), n.suc) = add_seq(partial(a), partial(b))(n.suc)
        }
    }
}

theorem partial_mul_seq_comm(a: Real, b: Nat -> Real) {
    partial(mul_seq(a, b)) = mul_seq(a, partial(b))
} by {
    // Show the sequences are pointwise equal
    forall(n: Nat) {
        mul_seq(a, b, n) = a * b(n)
        mul_fn(a, b, n) = a * b(n)
        mul_seq(a, b)(n) = mul_fn(a, b)(n)
    }
    mul_seq(a, b) = mul_fn(a, b)
    forall(n: Nat) {
        a * partial(b, n) = partial(mul_fn(a, b), n)
        mul_seq(a, partial(b), n) = a * partial(b, n)
        partial(mul_seq(a, b), n) = partial(mul_fn(a, b), n)
        partial(mul_seq(a, b), n) = mul_seq(a, partial(b), n)
    }
}

/// Scalar multiplication on the right of a partial sum.
theorem partial_mul_scalar_right(a: Nat -> Real, b: Real, n: Nat) {
    partial(a, n) * b = sum(map(n.range, mul_fn(b, a)))
} by {
}

define tail[T](a: Nat -> T, n: Nat, i: Nat) -> T {
    a(n + i)
}

theorem tail_zero[T](a: Nat -> T) {
    tail(a, Nat.0) = a
} by {
    forall(i: Nat) {
        tail(a, Nat.0, i) = a(Nat.0 + i)
        Nat.0 + i = i
        tail(a, Nat.0, i) = a(i)
    }
}

// Two different ways of adding the first m items, and the next n.
theorem partial_tail(a: Nat -> Real, m: Nat, n: Nat) {
    partial(a, m + n) = partial(a, m) + partial(tail(a, m), n)
} by {
    // Define a predicate for induction on n
    define p(k: Nat) -> Bool {
        partial(a, m + k) = partial(a, m) + partial(tail(a, m), k)
    }
    // Base case: n = 0
    m + Nat.0 = m
    partial(a, m) + partial(tail(a, m), Nat.0) = partial(a, m)
    p(Nat.0)

    // Inductive step
    forall(k: Nat) {
        if p(k) {
            // By the induction hypothesis

            // Now prove for k.suc
            // First, simplify the left side
            m + k.suc = (m + k).suc
            partial(a, m + k.suc) = partial(a, m + k) + a(m + k)

            // Substitute using induction hypothesis
            partial(a, m + k) = partial(a, m) + partial(tail(a, m), k)
            partial(a, m + k.suc) = partial(a, m) + partial(tail(a, m), k) + a(m + k)

            // For the right side
            partial(tail(a, m), k.suc) = partial(tail(a, m), k) + tail(a, m, k)
            tail(a, m, k) = a(m + k)
            partial(tail(a, m), k.suc) = partial(tail(a, m), k) + a(m + k)

            // Combine with partial(a, m)
            partial(a, m) + partial(tail(a, m), k.suc) = partial(a, m) + (partial(tail(a, m), k) + a(m + k))

            // By associativity

            // Therefore
            partial(a, m + k.suc) = partial(a, m) + partial(tail(a, m), k.suc)
            p(k.suc)
        }
    }
    p(n)
}

theorem partial_tail_sub(a: Nat -> Real, m: Nat, n: Nat) {
    partial(tail(a, m), n) = partial(a, m + n) - partial(a, m)
}

let const_seq = constant[Nat, Real]

theorem const_seq_n(a: Real, n: Nat) {
    const_seq(a)(n) = a
} by {
    const_seq(a) = constant[Nat, Real](a)
}

// The sequence-operations version of partial_tail.
// Partial and tail don't quite commute, there's an additive shift.
theorem tail_partial(a: Nat -> Real, n: Nat) {
    tail(partial(a), n) = add_seq(partial(tail(a, n)), const_seq(partial(a, n)))
} by {
    forall(i: Nat) {
        tail(partial(a), n, i) = partial(a, n + i)
        const_seq(partial(a, n), i) = partial(a, n)
        add_seq(partial(tail(a, n)), const_seq(partial(a, n)), i) = partial(tail(a, n), i) + const_seq(partial(a, n), i)
        partial(a, n) + partial(tail(a, n), i) = partial(tail(a, n), i) + partial(a, n)
        tail(partial(a), n, i) = add_seq(partial(tail(a, n)), const_seq(partial(a, n)), i)
    }
}

theorem partial_tail_rewrite(a: Nat -> Real, n: Nat) {
    partial(tail(a, n)) = add_seq(tail(partial(a), n), const_seq(-partial(a, n)))
} by {
    forall(i: Nat) {
        tail(partial(a), n, i) = partial(a, n + i)
        const_seq(-partial(a, n), i) = -partial(a, n)
        add_seq(tail(partial(a), n), const_seq(-partial(a, n)), i) = tail(partial(a), n, i) + const_seq(-partial(a, n), i)
        partial(a, n + i) + -partial(a, n) = partial(a, n + i) - partial(a, n)
        partial(tail(a, n), i) = add_seq(tail(partial(a), n), const_seq(-partial(a, n)), i)
    }
}

theorem tail_converges_to(a: Nat -> Real, n: Nat) {
    converges(a) implies
    converges_to(tail(a, n), limit(a))
} by {
    forall(eps: Real) {
        if eps.is_positive {
            let n1: Nat satisfy {
                tail_bound(a, limit(a), n1, eps)
            }
            let n2: Nat satisfy {
                n <= n2 and n1 <= n2
            }

            forall(i: Nat) {
                if n2 <= i {
                    i <= i + n
                    n2 <= i + n
                    n1 <= i + n
                    tail(a, n)(i) = a(n + i)
                    i + n = n + i
                    tail(a, n)(i).is_close(limit(a), eps)
                }
            }

            exists(k0: Nat) {
                n2 <= k0 and not tail(a, n, k0).is_close(limit(a), eps)
            } or tail_bound(tail(a, n), limit(a), n2, eps)
            tail_bound(tail(a, n), limit(a), n2, eps)
        }
    }

    // By the definition of converges_to, we've proven that tail(a, n) converges to limit(a)
}

theorem tail_imp_converges_to(a: Nat -> Real, n: Nat) {
    converges(tail(a, n)) implies
    converges_to(a, limit(tail(a, n)))
} by {
    forall(eps: Real) {
        if eps.is_positive {
            let n1: Nat satisfy {
                tail_bound(tail(a, n), limit(tail(a, n)), n1, eps)
            }
            let n2 = n + n1

            forall(i: Nat) {
                if n2 <= i {
                    let d: Nat satisfy {
                        n + n1 + d = i
                    }
                    n1 <= n1 + d
                    tail(a, n)(n1 + d).is_close(limit(tail(a, n)), eps)
                    n + (n1 + d) = i
                    tail(a, n)(n1 + d) = a(i)
                    a(i).is_close(limit(tail(a, n)), eps)
                }
            }

            exists(k0: Nat) {
                n2 <= k0 and not a(k0).is_close(limit(tail(a, n)), eps)
            } or tail_bound(a, limit(tail(a, n)), n2, eps)
            tail_bound(a, limit(tail(a, n)), n2, eps)
        }
    }
}

// Maybe we could just use tail_converges_to to prove this,
// along with sequence ops, and it would be simpler.
theorem tail_partial_converges(a: Nat -> Real, k: Nat) {
    converges(partial(a)) implies
    converges_to(partial(tail(a, k)), limit(partial(a)) - partial(a, k))
} by {
    let target = limit(partial(a)) - partial(a, k)
    forall(eps: Real) {
        if eps.is_positive {
            let n: Nat satisfy {
                tail_bound(partial(a), limit(partial(a)), n, eps)
            }
            forall(i: Nat) {
                if n <= i {
                    i <= i + k
                    n <= i + k
                    i + k = k + i
                    partial(a, k) + limit(partial(a)) - partial(a, k) = limit(partial(a))
                    partial(a, k + i).is_close(partial(a, k) + limit(partial(a)) - partial(a, k), eps)
                    partial(a, k) + partial(tail(a, k), i) = partial(a, k + i)
                    partial(a, k) + partial(tail(a, k), i) - partial(a, k) = partial(a, k) - partial(a, k) + partial(tail(a, k), i)
                    partial(a, k) + partial(tail(a, k), i) - (partial(a, k) + target) = partial(a, k) + partial(tail(a, k), i) - partial(a, k) - target
                    partial(a, k) + target = partial(a, k) + limit(partial(a)) - partial(a, k)
                    (partial(a, k) + partial(tail(a, k), i) - partial(a, k)).is_close(target, eps)
                    partial(tail(a, k), i).is_close(target, eps)
                }
            }
            tail_bound(partial(tail(a, k)), target, n, eps)
            exists(n0: Nat) {
                tail_bound(partial(tail(a, k)), target, n0, eps)
            }
        }
    }
    forall(eps: Real) {
        eps.is_positive implies exists(n0: Nat) {
            tail_bound(partial(tail(a, k)), target, n0, eps)
        }
    }
    exists(k0: Real) {
        k0.is_positive and forall(x0: Nat) {
            not tail_bound(partial(tail(a, k)), target, x0, k0)
        }
    } or converges_to(partial(tail(a, k)), target)
    converges_to(partial(tail(a, k)), target)
}

theorem converges_const_seq(a: Real) {
    converges(const_seq(a))
}

theorem add_seq_converges(a: Nat -> Real, b: Nat -> Real) {
    converges(a) and converges(b) implies converges(add_seq(a, b))
}

define neg_seq(a: Nat -> Real) -> (Nat -> Real) {
    mul_seq(-Real.1, a)
}

theorem neg_seq_neg_seq(a: Nat -> Real) {
    neg_seq(neg_seq(a)) = a
} by {
    forall(n: Nat) {
        mul_seq(-Real.1, a)(n) = -Real.1 * a(n)
        -Real.1 * a(n) = -a(n)
        mul_seq(-Real.1, neg_seq(a))(n) = -Real.1 * neg_seq(a)(n)
        -Real.1 * neg_seq(a)(n) = -neg_seq(a)(n)
        -neg_seq(a)(n) = --a(n)
        --a(n) = a(n)
        neg_seq(neg_seq(a))(n) = a(n)
    }
}

theorem neg_seq_lifts(a: Nat -> Rat) {
    neg_seq(lift_seq(a)) = lift_seq(neg_rat_seq(a))
} by {
    forall(n: Nat) {
        neg_seq(lift_seq(a), n) = mul_seq(-Real.1, lift_seq(a), n)
        mul_seq(-Real.1, lift_seq(a), n) = -Real.1 * lift_seq(a, n)
        lift_seq(a, n) = Real.from_rat(a(n))
        -Real.from_rat(a(n)) = Real.from_rat(-a(n))
        -a(n) = neg_rat_seq(a, n)
        lift_seq(neg_rat_seq(a), n) = Real.from_rat(neg_rat_seq(a, n))
        neg_seq(lift_seq(a), n) = lift_seq(neg_rat_seq(a), n)
    }
}

theorem neg_seq_cancels(a: Nat -> Real, b: Nat -> Real) {
    add_seq(add_seq(a, b), neg_seq(b)) = a
} by {
    forall(i: Nat) {
        add_seq(a, b, i) - b(i) = a(i)
        add_seq(add_seq(a, b), neg_seq(b))(i) = add_seq(a, b)(i) + neg_seq(b)(i)
        neg_seq(b)(i) = mul_seq(-Real.1, b)(i)
        mul_seq(-Real.1, b)(i) = -Real.1 * b(i)
        add_seq(a, b)(i) + -b(i) = add_seq(a, b)(i) - b(i)
        add_seq(add_seq(a, b), neg_seq(b))(i) = a(i)
    }
}

theorem neg_seq_converges_to(a: Nat -> Real) {
    converges(a) implies converges_to(neg_seq(a), -limit(a))
} by {
    neg_seq(a) = mul_seq(-Real.1, a)
}

theorem neg_seq_converges(a: Nat -> Real) {
    converges(a) implies converges(neg_seq(a))
}

theorem neg_seq_converges_converse(a: Nat -> Real) {
    converges(neg_seq(a)) implies converges(a)
} by {
    forall(n: Nat) {
        neg_seq(neg_seq(a), n) = mul_seq(-Real.1, neg_seq(a), n)
        mul_seq(-Real.1, neg_seq(a), n) = -Real.1 * neg_seq(a)(n)
        neg_seq(a)(n) = mul_seq(-Real.1, a, n)
        mul_seq(-Real.1, a, n) = -Real.1 * a(n)
        -Real.1 * neg_seq(a)(n) = -neg_seq(a)(n)
        -neg_seq(a)(n) = --a(n)
        --a(n) = a(n)
        neg_seq(neg_seq(a), n) = a(n)
    }
    neg_seq(neg_seq(a)) = a
}

/// A sequence is decreasing if every term is at most the preceding term.
define is_decreasing_seq(a: Nat -> Real) -> Bool {
    forall(n: Nat) {
        a(n.suc) <= a(n)
    }
}

/// Negation changes decreasing sequences into increasing sequences.
theorem decreasing_neg_increasing(a: Nat -> Real) {
    is_decreasing_seq(a) implies is_increasing(neg_seq(a))
} by {
    forall(n: Nat) {
        if is_decreasing_seq(a) {
            a(n.suc) <= a(n)
            -a(n) <= -a(n.suc)
            neg_seq(a, n) = -a(n)
            neg_seq(a, n.suc) = -a(n.suc)
            neg_seq(a, n) <= neg_seq(a, n.suc)
        }
    }
}

/// A decreasing sequence stays below each earlier term.
theorem distant_decreasing(a: Nat -> Real, m: Nat, n: Nat) {
    is_decreasing_seq(a) and m <= n implies a(n) <= a(m)
} by {
    if is_decreasing_seq(a) and m <= n {
        is_increasing(neg_seq(a))
        neg_seq(a, m) <= neg_seq(a, n)
        neg_seq(a, m) = -a(m)
        neg_seq(a, n) = -a(n)
        -a(m) <= -a(n)
        a(n) <= a(m)
    }
}

/// An antitone map from natural numbers to reals is a decreasing sequence.
theorem antitone_is_decreasing(a: Nat -> Real) {
    is_antitone(a) implies is_decreasing_seq(a)
} by {
    if is_antitone(a) {
        forall(n: Nat) {
            n <= n.suc
            a(n.suc) <= a(n)
        }
    }
}

/// A decreasing sequence is an antitone map from the naturals.
theorem decreasing_is_antitone(a: Nat -> Real) {
    is_decreasing_seq(a) implies is_antitone(a)
} by {
    if is_decreasing_seq(a) {
        forall(m: Nat, n: Nat) {
            if m <= n {
                distant_decreasing(a, m, n)
                a(n) <= a(m)
            }
        }
        antitone_from_forall(a)
        is_antitone(a)
    }
}

/// A local decreasing sequence and an antitone map from the naturals are the same condition.
theorem decreasing_iff_antitone(a: Nat -> Real) {
    is_decreasing_seq(a) = is_antitone(a)
} by {
    if is_decreasing_seq(a) {
        decreasing_is_antitone(a)
        is_antitone(a)
    }
    if is_antitone(a) {
        antitone_is_decreasing(a)
        is_decreasing_seq(a)
    }
    is_decreasing_seq(a) = is_antitone(a)
}

/// An antitone real sequence reverses order between arbitrary indices.
theorem distant_antitone(a: Nat -> Real, m: Nat, n: Nat) {
    is_antitone(a) and m <= n implies a(n) <= a(m)
} by {
    if is_antitone(a) and m <= n {
        is_decreasing_seq(a)
        distant_decreasing(a, m, n)
        a(n) <= a(m)
    }
}

/// The first term of an increasing sequence is a lower bound.
theorem increasing_lower_bound_first(a: Nat -> Real) {
    is_increasing(a) implies is_lower_bound(a, a(Nat.0))
} by {
    forall(n: Nat) {
        if is_increasing(a) {
            Nat.0 <= n
            a(Nat.0) <= a(n)
        }
    }
}

/// The first term of a monotone real sequence is a lower bound.
theorem monotone_lower_bound_first(a: Nat -> Real) {
    is_monotone(a) implies is_lower_bound(a, a(Nat.0))
} by {
    if is_monotone(a) {
        is_increasing(a)
        increasing_lower_bound_first(a)
        is_lower_bound(a, a(Nat.0))
    }
}

/// The first term of a decreasing sequence is an upper bound.
theorem decreasing_upper_bound_first(a: Nat -> Real) {
    is_decreasing_seq(a) implies is_upper_bound(a, a(Nat.0))
} by {
    forall(n: Nat) {
        if is_decreasing_seq(a) {
            Nat.0 <= n
            a(n) <= a(Nat.0)
        }
    }
}

/// The first term of an antitone real sequence is an upper bound.
theorem antitone_upper_bound_first(a: Nat -> Real) {
    is_antitone(a) implies is_upper_bound(a, a(Nat.0))
} by {
    if is_antitone(a) {
        is_decreasing_seq(a)
        decreasing_upper_bound_first(a)
        is_upper_bound(a, a(Nat.0))
    }
}

/// Negation changes increasing sequences into decreasing sequences.
theorem increasing_neg_decreasing(a: Nat -> Real) {
    is_increasing(a) implies is_decreasing_seq(neg_seq(a))
} by {
    forall(n: Nat) {
        if is_increasing(a) {
            a(n) <= a(n.suc)
            -a(n.suc) <= -a(n)
            neg_seq(a, n) = -a(n)
            neg_seq(a, n.suc) = -a(n.suc)
            neg_seq(a, n.suc) <= neg_seq(a, n)
        }
    }
}

/// A lower bound for a sequence is an upper bound for its negation.
theorem lower_bound_neg_upper_bound(a: Nat -> Real, lb: Real) {
    is_lower_bound(a, lb) implies is_upper_bound(neg_seq(a), -lb)
} by {
    forall(n: Nat) {
        if is_lower_bound(a, lb) {
            lb <= a(n)
            -a(n) <= -lb
            neg_seq(a, n) = -a(n)
            neg_seq(a, n) <= -lb
        }
    }
}

/// An upper bound for a sequence is a lower bound for its negation.
theorem upper_bound_neg_lower_bound(a: Nat -> Real, ub: Real) {
    is_upper_bound(a, ub) implies is_lower_bound(neg_seq(a), -ub)
} by {
    forall(n: Nat) {
        if is_upper_bound(a, ub) {
            a(n) <= ub
            -ub <= -a(n)
            neg_seq(a, n) = -a(n)
            -ub <= neg_seq(a, n)
        }
    }
}

/// A decreasing sequence with a lower bound converges.
theorem decreasing_bounded_below_converges(a: Nat -> Real, lb: Real) {
    is_decreasing_seq(a) and is_lower_bound(a, lb) implies converges(a)
} by {
    if is_decreasing_seq(a) and is_lower_bound(a, lb) {
        is_increasing(neg_seq(a))
        is_upper_bound(neg_seq(a), -lb)
        converges(neg_seq(a))
        converges(a)
    }
}

/// A decreasing sequence with a lower bound converges.
theorem decreasing_has_lower_bound_converges(a: Nat -> Real) {
    is_decreasing_seq(a) and has_lower_bound_seq(a) implies converges(a)
} by {
    if is_decreasing_seq(a) and has_lower_bound_seq(a) {
        let lb: Real satisfy {
            is_lower_bound(a, lb)
        }
        converges(a)
    }
}

/// An antitone real sequence with a lower bound converges.
theorem antitone_bounded_below_converges(a: Nat -> Real, lb: Real) {
    is_antitone(a) and is_lower_bound(a, lb) implies converges(a)
} by {
    if is_antitone(a) and is_lower_bound(a, lb) {
        is_decreasing_seq(a)
        converges(a)
    }
}

/// An antitone real sequence with some lower bound converges.
theorem antitone_has_lower_bound_converges(a: Nat -> Real) {
    is_antitone(a) and has_lower_bound_seq(a) implies converges(a)
} by {
    if is_antitone(a) and has_lower_bound_seq(a) {
        is_decreasing_seq(a)
        converges(a)
    }
}

/// The limit of a negated sequence is the negation of the limit.
theorem limit_neg_seq(a: Nat -> Real) {
    converges(a) implies limit(neg_seq(a)) = -limit(a)
} by {
    if converges(a) {
        converges(neg_seq(a))
        converges_to(neg_seq(a), -limit(a))
        converges_to(neg_seq(a), limit(neg_seq(a)))
        converges_to_unique(neg_seq(a), -limit(a), limit(neg_seq(a)))
        limit(neg_seq(a)) = -limit(a)
    }
}

/// A convergent decreasing sequence is bounded below by its limit.
theorem decreasing_convergent_bounded_by_limit(a: Nat -> Real) {
    is_decreasing_seq(a) and converges(a)
    implies is_lower_bound(a, limit(a))
} by {
    if is_decreasing_seq(a) and converges(a) {
        is_increasing(neg_seq(a))
        converges(neg_seq(a))
        is_upper_bound(neg_seq(a), limit(neg_seq(a)))
        limit(neg_seq(a)) = -limit(a)
        forall(n: Nat) {
            neg_seq(a, n) <= limit(neg_seq(a))
            neg_seq(a, n) = -a(n)
            -a(n) <= -limit(a)
            limit(a) <= a(n)
        }
    }
}

/// A convergent antitone real sequence is bounded below by its limit.
theorem antitone_convergent_bounded_by_limit(a: Nat -> Real) {
    is_antitone(a) and converges(a) implies is_lower_bound(a, limit(a))
} by {
    if is_antitone(a) and converges(a) {
        is_decreasing_seq(a)
        is_lower_bound(a, limit(a))
    }
}

theorem conv_add_imp_conv_right(a: Nat -> Real, b: Nat -> Real) {
    converges(add_seq(a, b)) and converges(a) implies converges(b)
} by {
    forall(i: Nat) {
        add_seq(a, b, i) = a(i) + b(i)
        add_seq(b, a, i) = b(i) + a(i)
        a(i) + b(i) = b(i) + a(i)
        add_seq(a, b, i) = add_seq(b, a, i)
    }
    add_seq(a, b) = add_seq(b, a)
    forall(i: Nat) {
        add_seq(add_seq(a, b), neg_seq(a), i) = add_seq(a, b)(i) + neg_seq(a)(i)
        add_seq(a, b)(i) = a(i) + b(i)
        neg_seq(a)(i) = mul_seq(-Real.1, a)(i)
        mul_seq(-Real.1, a)(i) = -Real.1 * a(i)
        add_seq(a, b)(i) + -a(i) = add_seq(a, b)(i) - a(i)
        add_seq(a, b, i) - a(i) = b(i)
        add_seq(add_seq(a, b), neg_seq(a), i) = b(i)
    }
    add_seq(add_seq(a, b), neg_seq(a)) = b
}

theorem conv_add_imp_conv_left(a: Nat -> Real, b: Nat -> Real) {
    converges(add_seq(a, b)) and converges(b) implies converges(a)
} by {
    forall(i: Nat) {
        add_seq(a, b, i) = a(i) + b(i)
        add_seq(b, a, i) = b(i) + a(i)
        a(i) + b(i) = b(i) + a(i)
        add_seq(a, b, i) = add_seq(b, a, i)
    }
    add_seq(a, b) = add_seq(b, a)
}

theorem partial_tail_conv_imp_partial_conv(a: Nat -> Real, n: Nat) {
    converges(partial(tail(a, n))) implies converges(partial(a))
} by {
    partial(tail(a, n)) = add_seq(tail(partial(a), n), const_seq(-partial(a, n)))
    converges(const_seq(-partial(a, n)))
    converges(tail(partial(a), n))
    converges_to(partial(a), limit(tail(partial(a), n)))
}

theorem triangle_ineq(a: Real, b: Real) {
    (a + b).abs <= a.abs + b.abs
} by {
    if (a + b).is_negative {

        // Show -a + -b <= a.abs + b.abs
        -a + -b <= -a + b.abs

        -a + b.abs <= a.abs + b.abs

        (a + b).abs <= a.abs + b.abs
    } else {

        a + b.abs <= a.abs + b.abs

    }
}

theorem sum_abs_le_abs_sum(items: List[Real]) {
    sum(items).abs <= sum(map(items, Real.abs))
} by {
    define p(xs: List[Real]) -> Bool {
        sum(xs).abs <= sum(map(xs, Real.abs))
    }

    // Base case
    sum(List.nil[Real]).abs <= sum(List.nil[Real]).abs
    map(List.nil[Real], Real.abs) = List.nil[Real]
    sum(List.nil[Real]).abs <= sum(map(List.nil[Real], Real.abs))
    p(List.nil)

    // Inductive step
    forall(head: Real, rest: List[Real]) {
        if p(rest) {
            head.abs + sum(rest).abs <= head.abs + sum(map(rest, Real.abs))
            (head + sum(rest)).abs <= head.abs + sum(rest).abs
            (head + sum(rest)).abs <= head.abs + sum(map(rest, Real.abs))
            head + sum(rest) = sum(List.cons(head, rest))
            List.cons(head.abs, map(rest, Real.abs)) = map(List.cons(head, rest), Real.abs)
            head.abs + sum(map(rest, Real.abs)) = sum(List.cons(head.abs, map(rest, Real.abs)))
            sum(List.cons(head, rest)).abs <= sum(map(List.cons(head, rest), Real.abs))
            p(List.cons(head, rest))
        }
    }
}

// Maybe we also want the abs version.
theorem diff_partial(a: Nat -> Real, m: Nat, n: Nat) {
    m <= n implies
    partial(a, n) - partial(a, m) = sum(map(m.until(n), a))
} by {
    m.range + m.until(n) = n.range
    map(m.range, a) + map(m.until(n), a) = map(m.range + m.until(n), a)
    sum(map(m.range, a)) + sum(map(m.until(n), a)) = sum(map(m.range, a) + map(m.until(n), a))
    sum(map(m.until(n), a)) + sum(map(m.range, a)) = sum(map(m.range, a)) + sum(map(m.until(n), a))
    sum(map(n.range, a)) - sum(map(m.range, a)) = sum(map(m.until(n), a))
}

// Prove diff_partial first so that we can simplify partial closeness.
theorem abs_conv_imp_conv(a: Nat -> Real) {
    converges(partial(compose(Real.abs, a)))
    implies
    converges(partial(a))
} by {
    let p = partial(a)
    let q = partial(compose(Real.abs, a))
    forall(eps: Real) {
        if eps.is_positive {
            let n: Nat satisfy {
                cauchy_bound(q, n, eps)
            }
            forall(i: Nat, j: Nat) {
                if n <= i and n <= j {
                    if i <= j {
                        q(j) - q(i) = sum(map(map(i.until(j), a), Real.abs))
                        partial(a, j) - partial(a, i) = sum(map(i.until(j), a))
                        sum(map(i.until(j), a)).abs <= sum(map(map(i.until(j), a), Real.abs))
                        q(j) - q(i) <= (q(j) - q(i)).abs
                        (p(j) - p(i)).abs <= (q(j) - q(i)).abs
                        (p(j) - p(i)).abs < eps
                        p(i).is_close(p(j), eps)
                    } else {
                        partial(compose(Real.abs, a), i) - partial(compose(Real.abs, a), j) = sum(map(j.until(i), compose(Real.abs, a)))
                        sum(map(j.until(i), a)).abs <= q(i) - q(j)
                        partial(a, i) - partial(a, j) = sum(map(j.until(i), a))
                        (p(i) - p(j)).abs <= (q(i) - q(j)).abs
                        q(i).is_close(q(j), eps)
                        (q(i) - q(j)).abs < eps
                        (p(i) - p(j)).abs < eps
                        p(i).is_close(p(j), eps)
                    }
                }
            }
            exists(k0: Nat, k1: Nat) {
                n <= k0 and n <= k1 and not p(k0).is_close(p(k1), eps)
            } or cauchy_bound(p, n, eps)
            cauchy_bound(p, n, eps)
            cauchy_bound(partial(a), n, eps)
        }
    }
}

/// If a series converges, its terms converge to zero.
theorem series_conv_imp_term_vanishes(a: Nat -> Real) {
    converges(partial(a)) implies converges_to(a, Real.0)
} by {
    let p = partial(a)
    forall(eps: Real) {
        if eps.is_positive {
            let n: Nat satisfy {
                cauchy_bound(p, n, eps)
            }
            forall(i: Nat) {
                if n <= i {
                    p(i).is_close(p(i.suc), eps)
                    p(i.suc) = p(i) + a(i)
                    p(i.suc) - p(i) = a(i)
                    (p(i.suc) - p(i)).abs < eps
                    a(i).abs < eps
                    a(i).is_close(Real.0, eps)
                }
            }
            tail_bound(a, Real.0, n, eps)
        }
    }
}

theorem pow_nonneg(r: Real, n: Nat) {
    0 <= r implies 0 <= r.pow(n)
} by {
    // Proof by induction
    define p(k: Nat) -> Bool {
        0 <= r.pow(k)
    }
    r.pow(Nat.0) = Real.1
    Real.1 * Real.1 = Real.1
    Real.1 * Real.1 >= Real.0
    Real.0 <= Real.1
    p(Nat.0)
    forall(k: Nat) {
        if p(k) {
            Real.0 <= r.pow(k)
            not r.is_negative
            not r.pow(k).is_negative
            r * r.pow(k) = r.pow(k.suc)
            0 <= r.pow(k.suc)
            p(k.suc)
        }
    }
    p(n)
}

theorem pos_geom_indirect_upper_bound(r: Real, n: Nat) {
    0 <= r and r < 1 implies
    partial(r.pow, n) * (1 - r) <= 1
} by {
    (r + -Real.1) * sum(map(n.range, r.pow)) = r.pow(n) + -Real.1
    Real.1 + -r = Real.1 - r
    --r = r
    sum(map(n.range, r.pow)) * (Real.1 - r) = (Real.1 - r) * sum(map(n.range, r.pow))
    Real.1 + -r.pow(n) = Real.1 - r.pow(n)
    --r.pow(n) = r.pow(n)
    sum(map(n.range, r.pow)) * (1 - r) = 1 - r.pow(n)
    let x = r.pow(n)
    0 + (1 - x) <= x + (1 - x)
    sum(map(n.range, r.pow)) * (1 - r) <= 1
}

theorem pos_mul_eq_pos(a: Real, b: Real) {
    a.is_positive and (a * b).is_positive implies b.is_positive
} by {
    a.abs = a
    (a * b).abs = a * b
    a.abs * b.abs = (a * b).abs
    a * b.abs = a * b
    if (-b).is_negative {
        b.is_positive
    } else {
        (-b).abs = -b
        (--b).abs = (-b).abs
        --b = b
        b.abs = -b
        a * -b = -(a * b)
        (a * b).is_negative
    }
}

theorem exists_large_mul(a: Real, b: Real) {
    a.is_positive and b.is_positive implies exists(c: Real) {
        a < b * c
    }
} by {
    let ra: Rat satisfy {
        a < Real.from_rat(ra)
    }
    Real.from_rat(ra).is_positive
    ra.is_positive
    let rb: Rat satisfy {
        rb.is_positive and Real.from_rat(rb) < b
    }
    let rc = ra / rb
    ra = rb * rc
    Real.from_rat(rb) * Real.from_rat(rc) < b * Real.from_rat(rc)
}

define nonneg_seq(a: Nat -> Real) -> Bool {
    forall(n: Nat) {
        not a(n).is_negative
    }
}

theorem nonneg_imp_partial_increasing(a: Nat -> Real) {
    nonneg_seq(a) implies is_increasing(partial(a))
}

theorem increasing_from_nonneg_start(a: Nat -> Real) {
    not a(Nat.0).is_negative and is_increasing(a) implies nonneg_seq(a)
} by {
    forall(n: Nat) {
        Nat.0 <= n
        Nat.0 <= Nat.0 + n
        Nat.0 + n = n
        a(Nat.0) <= a(n)
        not a(n).is_negative
    }
}

theorem nonneg_partial_nonneg(a: Nat -> Real) {
    nonneg_seq(a) implies
    nonneg_seq(partial(a))
}

/// Partial sums of nonnegative sequences are nonnegative.
theorem partial_nonneg(f: Nat -> Real, n: Nat) {
    is_lower_bound(f, Real.0) implies partial(f, n) >= Real.0
} by {

    define p(m: Nat) -> Bool {
        is_lower_bound(f, Real.0) implies partial(f, m) >= Real.0
    }

    p(Nat.0)

    forall(m: Nat) {
        if p(m) {
            if is_lower_bound(f, Real.0) {
                Real.0 <= partial(f, m) + f(m)
                partial(f, m) + f(m) = partial(f, m.suc)
                partial(f, m.suc) >= Real.0
            }
            p(m.suc)
        }
    }

    p(n)
}

theorem nonneg_pow(r: Real) {
    0 <= r implies nonneg_seq(r.pow)
}

theorem pos_geom_has_upper_bound(r: Real) {
    0 <= r and r < 1 implies
    exists(ub: Real) {
        is_upper_bound(partial(r.pow), ub)
    }
} by {
    (1 - r).is_positive
    1.is_positive
    let ub0: Real satisfy {
        1 < (1 - r) * ub0
    }
    (1 - r) * ub0 = ub0 * (1 - r)
    exists(ub: Real) {
        1 < ub * (1 - r)
    }
    let ub: Real satisfy {
        1 < ub * (1 - r)
    }
    ub.is_positive
    forall(n: Nat) {
        partial(r.pow, n) * (1 - r) * ub <= ub
        nonneg_seq(partial(r.pow))
        partial(r.pow, n) * 1 <= partial(r.pow, n) * ((1 - r) * ub)
        partial(r.pow, n) <= ub
    }
    is_upper_bound(partial(r.pow), ub)
    exists(ub2: Real) {
        is_upper_bound(partial(r.pow), ub2)
    }
}

theorem pos_geom_converges(r: Real) {
    0 <= r and r < 1 implies converges(partial(r.pow))
}

theorem abs_pow(r: Real, n: Nat) {
    r.abs.pow(n) = r.pow(n).abs
} by {
    // Proof by induction on n
    define p(k: Nat) -> Bool {
        r.abs.pow(k) = r.pow(k).abs
    }
    // Base case: n = 0
    r.abs.pow(Nat.0) = Real.1
    r.pow(Nat.0) = Real.1
    Real.1.is_positive
    Real.1.abs = Real.1
    r.abs.pow(Nat.0) = r.pow(Nat.0).abs
    p(Nat.0)

    // Inductive step
    forall(k: Nat) {
        if p(k) {
            // Induction hypothesis: r.abs.pow(k) = r.pow(k).abs

            // Need to prove: r.abs.pow(k.suc) = r.pow(k.suc).abs
            r.abs.pow(k.suc) = r.abs * r.abs.pow(k)
            r.pow(k.suc) = r * r.pow(k)
            r.abs * r.abs.pow(k) = r.abs * r.pow(k).abs
            r.abs * r.pow(k).abs = (r * r.pow(k)).abs
            r.abs.pow(k.suc) = r.pow(k.suc).abs
            p(k.suc)
        }
    }

    // By induction, p holds for all k
    p(n)
}

theorem abs_geom_comm(r: Real) {
    compose(Real.abs, r.pow) = r.abs.pow
} by {
    forall(n: Nat) {
        compose(Real.abs, r.pow, n) = r.pow(n).abs
        r.pow(n).abs = r.abs.pow(n)
        compose(Real.abs, r.pow, n) = r.abs.pow(n)
    }
}

theorem geom_converges(r: Real) {
    r.abs < 1 implies converges(partial(r.pow))
} by {
    if r.abs < 1 {
        Real.0 <= r.abs
        forall(n: Nat) {
            forall(k: Nat) {
                if k < n {
                    compose(Real.abs, r.pow, k) = r.pow(k).abs
                    r.abs.pow(k) = r.pow(k).abs
                    r.abs.pow(k) = compose(Real.abs, r.pow, k)
                }
            }
            partial(r.abs.pow, n) = partial(compose(Real.abs, r.pow), n)
        }
        partial(r.abs.pow) = partial(compose(Real.abs, r.pow))
        converges(partial(compose(Real.abs, r.pow)))
        converges(partial(r.pow))
    }
    forall(n: Nat) {
        forall(k: Nat) {
            if k < n {
                compose(Real.abs, r.pow, k) = r.pow(k).abs
                r.abs.pow(k) = r.pow(k).abs
                r.abs.pow(k) = compose(Real.abs, r.pow, k)
            }
        }
        partial(r.abs.pow, n) = partial(compose(Real.abs, r.pow), n)
    }
    partial(r.abs.pow) = partial(compose(Real.abs, r.pow))
}

theorem partial_tail_decomp(a: Nat -> Real, m: Nat) {
    converges(partial(a)) implies
    limit(partial(a)) = partial(a, m) + limit(partial(tail(a, m)))
} by {
    converges_to(partial(tail(a, m)), limit(partial(a)) - partial(a, m))
    converges(partial(tail(a, m)))
    converges_to(partial(tail(a, m)), limit(partial(tail(a, m))))
    limit(partial(tail(a, m))) = limit(partial(a)) - partial(a, m)
}

theorem pow_tail(r: Real) {
    tail(r.pow, Nat.1) = mul_seq(r, r.pow)
} by {
    forall(n: Nat) {
        tail(r.pow, Nat.1, n) = r.pow(Nat.1 + n)
        Nat.1 + n = n.suc
        r.pow(n.suc) = r * r.pow(n)
        mul_seq(r, r.pow, n) = r * r.pow(n)
        tail(r.pow, Nat.1, n) = mul_seq(r, r.pow, n)
    }
}

theorem partial_one(a: Nat -> Real) {
    partial(a, Nat.1) = a(Nat.0)
}

theorem geom_series_no_div(r: Real) {
    r.abs < 1 implies
    1 + r * limit(partial(r.pow)) = limit(partial(r.pow))
} by {
    partial(r.pow, Nat.1) = Real.1
    let t = tail(r.pow, Nat.1)
    forall(n: Nat) {
        forall(k: Nat) {
            if k < n {
                tail(r.pow, Nat.1, k) = r.pow(Nat.1 + k)
                Nat.1 + k = k.suc
                r.pow(k.suc) = r * r.pow(k)
                mul_seq(r, r.pow, k) = r * r.pow(k)
                tail(r.pow, Nat.1, k) = mul_seq(r, r.pow, k)
            }
        }
        partial(tail(r.pow, Nat.1), n) = partial(mul_seq(r, r.pow), n)
    }
    partial(tail(r.pow, Nat.1)) = partial(mul_seq(r, r.pow))
    partial(mul_seq(r, r.pow)) = mul_seq(r, partial(r.pow))
    converges(partial(r.pow))
    converges(mul_seq(r, partial(r.pow)))
    limit(partial(mul_seq(r, r.pow))) = r * limit(partial(r.pow))
}

/// Delta function: returns v if k = a, and 0 otherwise.
define delta(v: Real, a: Nat, k: Nat) -> Real {
    if k = a {
        v
    } else {
        Real.0
    }
}

/// For indices less than a, delta is zero.
theorem delta_before(v: Real, a: Nat, k: Nat) {
    k < a implies delta(v, a, k) = Real.0
} by {
    if k < a {
        k != a
        delta(v, a, k) = Real.0
    }
}

/// At index a, delta returns v.
theorem delta_at(v: Real, a: Nat) {
    delta(v, a, a) = v
} by {
    a = a
    delta(v, a, a) = v
}

/// For indices greater than a, delta is zero.
theorem delta_after(v: Real, a: Nat, k: Nat) {
    k > a implies delta(v, a, k) = Real.0
} by {
    if k > a {
        k != a
        delta(v, a, k) = Real.0
    }
}

/// Delta as a sequence function.
define delta_seq(v: Real, a: Nat) -> (Nat -> Real) {
    function(k: Nat) {
        delta(v, a, k)
    }
}

/// Partial sums of delta before reaching a.
theorem partial_delta_before(v: Real, a: Nat, n: Nat) {
    n <= a implies partial(delta_seq(v, a), n) = Real.0
} by {
    define p(m: Nat) -> Bool {
        m <= a implies partial(delta_seq(v, a), m) = Real.0
    }

    // Base case: n = 0
    p(Nat.0)

    // Inductive step
    forall(m: Nat) {
        if p(m) {
            if m.suc <= a {
                // By IH: partial(delta_seq(v, a), m) = 0
                partial(delta_seq(v, a), m) = Real.0

                // Also m < a, so delta(v, a, m) = 0
                m < m.suc
                m.suc <= a
                m < a
                delta(v, a, m) = Real.0
                delta_seq(v, a)(m) = Real.0

                // Therefore partial(delta_seq(v, a), m.suc) = 0 + 0 = 0
                partial(delta_seq(v, a), m.suc) = partial(delta_seq(v, a), m) + delta_seq(v, a)(m)
                partial(delta_seq(v, a), m.suc) = Real.0
            }
            p(m.suc)
        }
    }

    p(n)
}

/// Partial sums of delta at or after a equal v.
theorem partial_delta_at_or_after(v: Real, a: Nat, n: Nat) {
    a < n implies partial(delta_seq(v, a), n) = v
} by {
    if a < n {
        // We can write n = a.suc + k for some k
        let k: Nat satisfy {
            a.suc + k = n
        }

        // partial(delta_seq(v, a), a.suc + k)
        // = partial(delta_seq(v, a), a.suc) + partial(tail(delta_seq(v, a), a.suc), k)
        partial(delta_seq(v, a), a.suc + k) = partial(delta_seq(v, a), a.suc) + partial(tail(delta_seq(v, a), a.suc), k)

        // partial(delta_seq(v, a), a.suc) = partial(delta_seq(v, a), a) + delta(v, a, a)
        partial(delta_seq(v, a), a.suc) = partial(delta_seq(v, a), a) + delta_seq(v, a)(a)
        delta_seq(v, a)(a) = delta(v, a, a)

        // partial(delta_seq(v, a), a) = 0 (by partial_delta_before)
        a <= a
        partial(delta_seq(v, a), a) = Real.0

        // delta(v, a, a) = v
        delta(v, a, a) = v

        // So partial(delta_seq(v, a), a.suc) = 0 + v = v
        partial(delta_seq(v, a), a.suc) = v

        // For the tail: all elements are 0
        forall(j: Nat) {
            tail(delta_seq(v, a), a.suc)(j) = delta_seq(v, a)(a.suc + j)
            delta_seq(v, a)(a.suc + j) = delta(v, a, a.suc + j)
            a.suc + j > a
            delta(v, a, a.suc + j) = Real.0
            tail(delta_seq(v, a), a.suc)(j) = Real.0
        }
        forall(j: Nat) {
            j < k implies tail(delta_seq(v, a), a.suc)(j) = Real.0
        }

        // So partial(tail(delta_seq(v, a), a.suc), k) = 0
        partial(tail(delta_seq(v, a), a.suc), k) = Real.0

        // Therefore partial(delta_seq(v, a), n) = v + 0 = v
        partial(delta_seq(v, a), n) = v
    }
}

/// The partial sums of delta converge to v.
theorem delta_partial_converges(v: Real, a: Nat) {
    converges_to(partial(delta_seq(v, a)), v)
} by {
    // For all n > a, partial(delta_seq(v, a), n) = v
    // So the tail starting from a.suc is the constant sequence v
    forall(i: Nat) {
        a < a.suc + i
        partial(delta_seq(v, a), a.suc + i) = v
        tail(partial(delta_seq(v, a)), a.suc)(i) = partial(delta_seq(v, a))(a.suc + i)
        partial(delta_seq(v, a))(a.suc + i) = partial(delta_seq(v, a), a.suc + i)
        tail(partial(delta_seq(v, a)), a.suc)(i) = v
        constant(v, i) = v
        tail(partial(delta_seq(v, a)), a.suc)(i) = constant(v, i)
    }

    tail(partial(delta_seq(v, a)), a.suc) = constant[Nat, Real](v)
    converges_to(constant[Nat, Real](v), v)
    converges_to(tail(partial(delta_seq(v, a)), a.suc), v)
    converges_to(partial(delta_seq(v, a)), v)
}

/// Helper: absolute value of delta equals delta of absolute value.
theorem delta_abs_commute(v: Real, a: Nat, k: Nat) {
    delta(v, a, k).abs = delta(v.abs, a, k)
} by {
    if k = a {
        delta(v, a, k) = v
        delta(v.abs, a, k) = v.abs
        delta(v, a, k).abs = delta(v.abs, a, k)
    }
    if k != a {
        delta(v, a, k) = Real.0
        delta(v.abs, a, k) = Real.0
        delta(v, a, k).abs = delta(v.abs, a, k)
    }
}

/// The partial sums of delta absolutely converge.
theorem delta_absolutely_converges(v: Real, a: Nat) {
    converges(partial(compose(Real.abs, delta_seq(v, a))))
} by {
    // Show that compose(Real.abs, delta_seq(v, a)) = delta_seq(v.abs, a)
    forall(k: Nat) {
        compose(Real.abs, delta_seq(v, a))(k) = delta_seq(v, a)(k).abs
        delta_seq(v, a)(k) = delta(v, a, k)
        delta_seq(v, a)(k).abs = delta(v, a, k).abs
        delta(v, a, k).abs = delta(v.abs, a, k)
        delta(v.abs, a, k) = delta_seq(v.abs, a)(k)
        compose(Real.abs, delta_seq(v, a))(k) = delta_seq(v.abs, a)(k)
    }
    compose(Real.abs, delta_seq(v, a)) = delta_seq(v.abs, a)

    // We've proven that delta_partial_converges, so:
    converges_to(partial(delta_seq(v.abs, a)), v.abs)
    converges(partial(delta_seq(v.abs, a)))
    converges(partial(compose(Real.abs, delta_seq(v, a))))
}

/// Alias endpoint: partial sums split into an initial block plus a tail block.
theorem partial_sum_split_tail(a: Nat -> Real, m: Nat, n: Nat) {
    partial(a, m + n) = partial(a, m) + partial(tail(a, m), n)
} by {
    partial_tail(a, m, n)
}

/// Alias endpoint: a tail block is the corresponding partial-sum difference.
theorem tail_partial_sum_sub(a: Nat -> Real, m: Nat, n: Nat) {
    partial(tail(a, m), n) = partial(a, m + n) - partial(a, m)
} by {
    partial_tail_sub(a, m, n)
}

/// Alias endpoint: if two term sequences are pointwise ordered then their partial sums are pointwise ordered.
theorem partial_sums_seq_lte(a: Nat -> Real, b: Nat -> Real) {
    seq_lte(a, b) implies seq_lte(partial(a), partial(b))
} by {
    partial_seq_lte(a, b)
}

/// Alias endpoint: comparison test for a dominated nonnegative series.
theorem series_comparison_converges(a: Nat -> Real, b: Nat -> Real) {
    is_lower_bound(a, Real.0)
    and seq_lte(a, b)
    and converges(partial(b))
    implies converges(partial(a))
} by {
    comparison_test(a, b)
}

/// Alias endpoint: partial sums of a nonnegative sequence are monotone.
theorem nonnegative_series_partials_monotone(a: Nat -> Real) {
    is_lower_bound(a, Real.0) implies is_monotone(partial(a))
} by {
    nonneg_partial_monotone(a)
}

/// Alias endpoint: a convergent series decomposes into a finite prefix and a tail limit.
theorem series_limit_split_tail(a: Nat -> Real, m: Nat) {
    converges(partial(a)) implies
    limit(partial(a)) = partial(a, m) + limit(partial(tail(a, m)))
} by {
    partial_tail_decomp(a, m)
}
