from rat import Rat
from real.real_base import is_cut, is_lower, is_greatest, has_greatest, is_dedekind_cut
from real.real_ring import Real

// This file proves theorems about sets of real numbers.

attributes Real {
    /// True if this real number is an upper bound for the set s.
    define is_set_upper_bound(self, s: Real -> Bool) -> Bool {
        forall(y: Real) {
            s(y) implies y <= self
        }
    }

    /// True if this real number is a lower bound for the set s.
    define is_set_lower_bound(self, s: Real -> Bool) -> Bool {
        forall(y: Real) {
            s(y) implies self <= y
        }
    }

    /// True if this real number is the least upper bound (supremum) for the set s.
    define is_set_least_upper_bound(self, s: Real -> Bool) -> Bool {
        self.is_set_upper_bound(s) and
        forall(y: Real) {
            y.is_set_upper_bound(s) implies self <= y
        }
    }

    /// True if this real number is the greatest lower bound (infimum) for the set s.
    define is_set_greatest_lower_bound(self, s: Real -> Bool) -> Bool {
        self.is_set_lower_bound(s) and
        forall(y: Real) {
            y.is_set_lower_bound(s) implies y <= self
        }
    }
}

define is_nonempty_pred(s: Real -> Bool) -> Bool {
    exists(x: Real) {
        s(x)
    }
}

// The supremum condition on s is that it's nonempty, and has
// an upper bound.
// sup_cut is the Dedekind cut that will be the supremum of s.
// First we define it, then we prove that it is a Dedekind cut.
define sup_cut(s: Real -> Bool, r: Rat) -> Bool {
    not Real.from_rat(r).is_set_upper_bound(s)
}

theorem sup_cut_is_cut(s: Real -> Bool, ub: Real) {
    is_nonempty_pred(s) and ub.is_set_upper_bound(s)
    implies
    is_cut(sup_cut(s))
} by {
    let x: Real satisfy {
        s(x)
    }

    let r1: Rat satisfy {
        Real.from_rat(r1) < x
    }
    x > Real.from_rat(r1)
    not Real.from_rat(r1).is_set_upper_bound(s)
    sup_cut(s, r1)

    let r2: Rat satisfy {
        Real.from_rat(r2) > ub
    }

    // Show that r2 is an upper bound of s
    forall(y: Real) {
        if s(y) {
            y <= Real.from_rat(r2)
        }
    }
    Real.from_rat(r2).is_set_upper_bound(s)
    not sup_cut(s, r2)
}

theorem sup_cut_is_lower(s: Real -> Bool, ub: Real) {
    is_nonempty_pred(s) and ub.is_set_upper_bound(s)
    implies
    is_lower(sup_cut(s))
} by {
    forall(x: Rat, y: Rat) {
        if sup_cut(s, y) and x < y {
            not Real.from_rat(y).is_set_upper_bound(s)
            exists(z0: Real) {
                s(z0) and not z0 <= Real.from_rat(y)
            }
            let z0: Real satisfy {
                s(z0) and not z0 <= Real.from_rat(y)
            }
            z0 > Real.from_rat(y)
            exists(z: Real) {
                s(z) and z > Real.from_rat(y)
            }
            let z: Real satisfy {
                s(z) and z > Real.from_rat(y)
            }
            Real.from_rat(y) < z
            Real.from_rat(y) <= z
            z > Real.from_rat(x)
            not Real.from_rat(x).is_set_upper_bound(s)
            sup_cut(s, x)
        }
    }
    forall(x: Rat, y: Rat) {
        sup_cut(s, y) and x < y implies sup_cut(s, x)
    }
    is_lower(sup_cut(s))
}

theorem sup_cut_not_has_greatest(s: Real -> Bool, ub: Real) {
    is_nonempty_pred(s) and ub.is_set_upper_bound(s)
    implies
    not has_greatest(sup_cut(s))
} by {
    if has_greatest(sup_cut(s)) {
        let q: Rat satisfy {
            is_greatest(sup_cut(s), q)
        }

        not Real.from_rat(q).is_set_upper_bound(s)
        exists(z0: Real) {
            s(z0) and not z0 <= Real.from_rat(q)
        }
        let z0: Real satisfy {
            s(z0) and not z0 <= Real.from_rat(q)
        }
        z0 > Real.from_rat(q)
        exists(z: Real) {
            s(z) and z > Real.from_rat(q)
        }
        let z: Real satisfy {
            s(z) and z > Real.from_rat(q)
        }

        Real.from_rat(q) < z
        let eps: Rat satisfy {
            eps.is_positive and Real.from_rat(q) + Real.from_rat(eps) < z
        }
        q < q + eps
        Real.from_rat(q + eps) < z
        let q_prime: Rat satisfy {
            q < q_prime and Real.from_rat(q_prime) < z
        }

        // Since Real.from_rat(q_prime) < z and z is in s,
        // Real.from_rat(q_prime) is not an upper bound for s
        // Therefore q_prime is in the cut.
        // But q < q_prime, contradicting our assumption.
        not q_prime <= q
        false
    }
}

theorem sup_cut_is_dedekind_cut(s: Real -> Bool, ub: Real) {
    is_nonempty_pred(s) and ub.is_set_upper_bound(s)
    implies
    is_dedekind_cut(sup_cut(s))
}

theorem ub_imp_lub(s: Real -> Bool, x: Real) {
    is_nonempty_pred(s) and x.is_set_upper_bound(s)
    implies exists(y: Real) {
        y.is_set_least_upper_bound(s)
    }
} by {
    is_dedekind_cut(sup_cut(s))
    let Option.some(y) = Real.new(sup_cut(s))

    // Part 1: Show y is an upper bound of s
    forall(z: Real) {
        if s(z) {
            if z > y {
                let q: Rat satisfy {
                    z > Real.from_rat(q) and Real.from_rat(q) > y
                }
                y.gt_rat(q) = sup_cut(s, q)
                Real.from_rat(q) > y
                not (y > Real.from_rat(q))
                not sup_cut(s, q)
                Real.from_rat(q).is_set_upper_bound(s)
                false
            }
        }
    }
    y.is_set_upper_bound(s)

    // Part 2: Show y is the least upper bound
    forall(w: Real) {
        if w.is_set_upper_bound(s) {
            // We'll prove y <= w by contradiction
            if y > w {
                let q: Rat satisfy {
                    y > Real.from_rat(q) and Real.from_rat(q) > w
                }
                not Real.from_rat(q).is_set_upper_bound(s)
                exists(z0: Real) {
                    s(z0) and not z0 <= Real.from_rat(q)
                }
                let z0: Real satisfy {
                    s(z0) and not z0 <= Real.from_rat(q)
                }
                z0 > Real.from_rat(q)
                exists(z: Real) {
                    s(z) and z > Real.from_rat(q)
                }
                let z: Real satisfy {
                    s(z) and z > Real.from_rat(q)
                }
                false
            }
        }
    }
    y.is_set_least_upper_bound(s)
    exists(y2: Real) {
        y2.is_set_least_upper_bound(s)
    }
}

define flip(f: Real -> Bool, x: Real) -> Bool {
    f(-x)
}

theorem flip_flip(f: Real -> Bool) {
    flip(flip(f)) = f
} by {
    forall(x: Real) {
        flip(flip(f), x) = flip(f, -x)
        flip(f, -x) = f(--x)
        --x = x
        flip(flip(f), x) = f(x)
    }
}

theorem lb_imp_flip_ub(f: Real -> Bool, x: Real) {
    x.is_set_lower_bound(f) implies (-x).is_set_upper_bound(flip(f))
} by {
    if x.is_set_lower_bound(f) {
        forall(y: Real) {
            if flip(f, y) {
                f(-y)
                x <= -y
                --y <= -x
                y <= -x
            }
        }
        (-x).is_set_upper_bound(flip(f))
    }
}

theorem neg_lte(a: Real, b: Real) {
    -a <= b implies -b <= a
} by {
    Real.0 + -b <= a + b + -b
}

theorem ub_imp_flip_lb(f: Real -> Bool, x: Real) {
    x.is_set_upper_bound(f) implies (-x).is_set_lower_bound(flip(f))
} by {
    forall(y: Real) {
        if flip(f, y) {
            f(-y)
            -y <= x
            -x <= y
        }
    }
}

theorem glb_imp_flip_lub(f: Real -> Bool, x: Real) {
    x.is_set_greatest_lower_bound(f)
    implies (-x).is_set_least_upper_bound(flip(f))
} by {
    x.is_set_lower_bound(f)
    (-x).is_set_upper_bound(flip(f))

    forall(y: Real) {
        if y.is_set_upper_bound(flip(f)) {
            forall(z: Real) {
                if f(z) {
                    flip(f, -z)
                    -z <= y
                    -y <= z
                }
            }
            (-y).is_set_lower_bound(f)
            -y <= x
            -x <= y
        }
    }
}

theorem lub_imp_flip_glb(f: Real -> Bool, x: Real) {
    x.is_set_least_upper_bound(f)
    implies (-x).is_set_greatest_lower_bound(flip(f))
} by {
    x.is_set_upper_bound(f)
    (-x).is_set_lower_bound(flip(f))

    forall(y: Real) {
        if y.is_set_lower_bound(flip(f)) {
            (-y).is_set_upper_bound(flip(flip(f)))
            forall(z: Real) {
                flip(flip(f), z) = flip(f, -z)
                flip(f, -z) = f(--z)
                --z = z
                flip(flip(f), z) = f(z)
            }
            flip(flip(f)) = f
            (-y).is_set_upper_bound(f)
            x <= -y
            y <= -x
        }
    }
}

theorem lb_imp_glb(s: Real -> Bool, x: Real) {
    is_nonempty_pred(s) and x.is_set_lower_bound(s)
    implies exists(y: Real) {
        y.is_set_greatest_lower_bound(s)
    }
} by {
    let z: Real satisfy {
        s(z)
    }
    flip(s, -z)
    is_nonempty_pred(flip(s))
    (-x).is_set_upper_bound(flip(s))
    exists(y: Real) {
        y.is_set_least_upper_bound(flip(s))
    }
    let y: Real satisfy {
        y.is_set_least_upper_bound(flip(s))
    }
    (-y).is_set_greatest_lower_bound(flip(flip(s)))
    forall(w: Real) {
        flip(flip(s), w) = flip(s, -w)
        flip(s, -w) = s(--w)
        --w = w
        flip(flip(s), w) = s(w)
    }
    flip(flip(s)) = s
    (-y).is_set_greatest_lower_bound(s)
    exists(y2: Real) {
        y2.is_set_greatest_lower_bound(s)
    }
}
