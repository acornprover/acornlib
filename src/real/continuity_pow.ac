from data.basic.function_algebra import pointwise_mul
from data.basic.functions import function_extensionality, identity_fn
from real.continuity_composition import constant_function_is_continuous
from real.continuity_pointwise_mul import continuous_pointwise_mul
from real.continuity_sequences import identity_function_is_continuous
from nat import Nat, alt_induction
from real.continuity_base import Real, continuous
from real.exp import pow_suc

/// The function raising each real to the fixed natural-number power n.
define pow_real_fn(n: Nat, x: Real) -> Real {
    x.pow(n)
}

/// Raising to the zeroth power agrees with the constant function one.
theorem pow_real_fn_zero {
    pow_real_fn(Nat.0) = constant[Real, Real](Real.1)
} by {
    forall(x: Real) {
        pow_real_fn(Nat.0, x) = x.pow(Nat.0)
        x.pow(Nat.0) = Real.1
        constant[Real, Real](Real.1, x) = Real.1
        pow_real_fn(Nat.0, x) = constant[Real, Real](Real.1, x)
    }
    function_extensionality(pow_real_fn(Nat.0), constant[Real, Real](Real.1))
}

/// Raising to the successor power agrees with the identity times the lower power.
theorem pow_real_fn_suc(n: Nat) {
    pow_real_fn(n.suc) = pointwise_mul[Real, Real](identity_fn[Real], pow_real_fn(n))
} by {
    forall(x: Real) {
        pow_real_fn(n.suc, x) = x.pow(n.suc)
        pow_suc(x, n)
        x.pow(n.suc) = x * x.pow(n)
        identity_fn[Real](x) = x
        pow_real_fn(n, x) = x.pow(n)
        pointwise_mul[Real, Real](identity_fn[Real], pow_real_fn(n), x) = identity_fn[Real](x) * pow_real_fn(n, x)
        pointwise_mul[Real, Real](identity_fn[Real], pow_real_fn(n), x) = x * x.pow(n)
        pow_real_fn(n.suc, x) = pointwise_mul[Real, Real](identity_fn[Real], pow_real_fn(n), x)
    }
    function_extensionality(pow_real_fn(n.suc), pointwise_mul[Real, Real](identity_fn[Real], pow_real_fn(n)))
}

/// Every natural-number power function on the reals is continuous.
theorem continuous_pow_real_fn(n: Nat) {
    continuous(pow_real_fn(n))
} by {
    define p(x: Nat) -> Bool { continuous(pow_real_fn(x)) }

    // Base case: the zeroth power is the constant function one.
    pow_real_fn_zero
    constant_function_is_continuous(Real.1)
    continuous(constant[Real, Real](Real.1))
    continuous(pow_real_fn(Nat.0))
    p(Nat.0)

    // Inductive step.
    forall(x: Nat) {
        if p(x) {
            continuous(pow_real_fn(x))
            identity_function_is_continuous
            continuous(identity_fn[Real])
            continuous_pointwise_mul(identity_fn[Real], pow_real_fn(x))
            continuous(pointwise_mul[Real, Real](identity_fn[Real], pow_real_fn(x)))
            pow_real_fn_suc(x)
            continuous(pow_real_fn(x.suc))
            p(x.suc)
        }
    }

    p(Nat.0) and forall(x: Nat) { p(x) implies p(x.suc) }
    alt_induction(p)
    forall(x: Nat) { p(x) }
    p(n)
}
