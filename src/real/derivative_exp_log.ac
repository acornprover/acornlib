/// Derivatives of the exponential and logarithm functions.

from nat import Nat, from_nat, from_nat_add
from rat import Rat
from order import lte_trans, not_lt_imp_gte, lt_of_lt_of_lte, lt_of_lte_of_lt
from list import partial, sum, map
from real.real_field import Real, div_le_of_mul_le, inverse_div
from real.exp import exp_term, exp_add, exp_term_partial_converges, exp_pos, exp_term_zero_index, factorial_pos, pow_suc, mul_frac_assoc, exp_zero, two, two_positive, zero_pow_pos, exp_increasing, exp_gt_one_plus_x, exp_term_abs, exp_term_nonzero, exp_term_always_nonzero, exp_term_abs_converges, factorial_suc_real, suc_pos, inverse_pos
from real.log import log_value, log_some_of_pos_exists, exp_log_or_zero, exp_injective, exp_neg
from real.real_seq import converges_to, tail_bound, eps_smaller_than_both, close_and_lt_imp_close, converges_to_unique, convergent_converges_to_limit
from real.real_series import tail, partial_tail_sub, partial_mul_seq_comm, mul_seq, mul_seq_converges_to, add_seq, add_seq_converges, neg_seq, neg_seq_converges_to, neg_seq_converges, const_converges, const_seq, const_limit, seq_lte, seq_lte_preserves_limit, tail_converges_to, converges_mul_seq, is_lower_bound, pow_nonneg, pos_geom_indirect_upper_bound, sum_abs_le_abs_sum, partial_seq_lte
from real.real_seq import limit_add_seq
from real.abs_conv import absolutely_converges, abs_fn, abs_fn_nonneg, absolutely_converges_comparison, partial_abs_fn_eq, abs_fn_tail_comm
from real.limits import limit_abs_seq, abs_seq
from real.harmonic import real_recip_antitone_pos, rat_from_nat_lte_of_nat_lte, real_inverse_antitone_pos_strict
from real.derivative_basic import has_derivative_at, difference_quotient, sub_ne_zero_of_ne
from real.derivative_continuity import div_mul_cancel_denominator
from real.derivative_quotient import reciprocal_real, reciprocal_real_apply
from real.continuity_base import continuous_at, continuous_condition
from real.continuity_reciprocal_div import continuous_at_reciprocal_real
from real.calculus_api import is_derivative_fn
from real.real_ring import exists_small_mul_variant_2, lt_mul_pos_left, mul_abs, lte_mul_nonneg_right, converges, limit, mul_le_mul_nonneg, from_nat_is_from_rat, mul_pos_pos
from real.real_base import abs_gte_zero, abs_not_neg, lte_lt_trans, lt_trans, self_close, pos_imp_eq_abs, lte_add_right, add_comm, abs_neg, lte_abs, lt_add_right, lt_add_left, pos_gt_zero, gt_zero_imp_pos
from real.derivative_rules import div_add_distrib

/// The shifted exponential term x^n / (n+1)!.
/// These are the terms of the series for (x.exp - 1) / x.
define exp_shift_term(x: Real, n: Nat) -> Real {
    x.pow(n) / Real.from_rat(Rat.from_nat(n.suc.factorial))
}

/// Multiplying a shifted term by x shifts it into the exponential series.
/// x * (x^n / (n+1)!) = x^(n+1) / (n+1)!.
theorem exp_shift_term_mul_x(x: Real, n: Nat) {
    x * exp_shift_term(x, n) = exp_term(x, n.suc)
} by {
    exp_shift_term(x, n) = x.pow(n) / Real.from_rat(Rat.from_nat(n.suc.factorial))
    x * exp_shift_term(x, n) = x * (x.pow(n) / Real.from_rat(Rat.from_nat(n.suc.factorial)))
    mul_frac_assoc(x, x.pow(n), Real.from_rat(Rat.from_nat(n.suc.factorial)))
    x * (x.pow(n) / Real.from_rat(Rat.from_nat(n.suc.factorial))) =
        (x * x.pow(n)) / Real.from_rat(Rat.from_nat(n.suc.factorial))
    pow_suc(x, n)
    x.pow(n.suc) = x * x.pow(n)
    (x * x.pow(n)) / Real.from_rat(Rat.from_nat(n.suc.factorial)) =
        x.pow(n.suc) / Real.from_rat(Rat.from_nat(n.suc.factorial))
    exp_term(x, n.suc) = x.pow(n.suc) / Real.from_rat(Rat.from_nat(n.suc.factorial))
    x * exp_shift_term(x, n) = exp_term(x, n.suc)
}

/// The shifted term sequence is the exponential sequence shifted by one index,
/// after multiplication by x.
theorem mul_seq_shift_eq_tail(x: Real) {
    mul_seq(x, exp_shift_term(x)) = tail(exp_term(x), Nat.1)
} by {
    forall(n: Nat) {
        mul_seq(x, exp_shift_term(x), n) = x * exp_shift_term(x, n)
        exp_shift_term_mul_x(x, n)
        x * exp_shift_term(x, n) = exp_term(x, n.suc)
        tail(exp_term(x), Nat.1, n) = exp_term(x, Nat.1 + n)
        Nat.1 + n = n.suc
        tail(exp_term(x), Nat.1, n) = exp_term(x, n.suc)
        mul_seq(x, exp_shift_term(x), n) = tail(exp_term(x), Nat.1, n)
    }
}

/// The partial sums of the shifted series relate to the exponential partial sums:
/// x * partial(exp_shift_term(x), n) = partial(exp_term(x), n+1) - 1.
theorem exp_partial_shift_mul(x: Real, n: Nat) {
    x * partial(exp_shift_term(x), n) = partial(exp_term(x), n.suc) - Real.1
} by {
    mul_seq_shift_eq_tail(x)
    mul_seq(x, exp_shift_term(x)) = tail(exp_term(x), Nat.1)
    partial_mul_seq_comm(x, exp_shift_term(x))
    partial(mul_seq(x, exp_shift_term(x))) = mul_seq(x, partial(exp_shift_term(x)))
    partial(mul_seq(x, exp_shift_term(x)), n) = x * partial(exp_shift_term(x), n)
    partial(tail(exp_term(x), Nat.1), n) =
        partial(exp_term(x), Nat.1 + n) - partial(exp_term(x), Nat.1)
    Nat.1 + n = n.suc
    partial(exp_term(x), Nat.1 + n) = partial(exp_term(x), n.suc)
    exp_term_zero_index(x)
    exp_term(x, Nat.0) = Real.1
    partial(exp_term(x), Nat.1) = exp_term(x, Nat.0)
    partial(exp_term(x), Nat.1) = Real.1
    partial(tail(exp_term(x), Nat.1), n) = partial(exp_term(x), n.suc) - Real.1
    x * partial(exp_shift_term(x), n) = partial(exp_term(x), n.suc) - Real.1
}

/// The factorial of a successor is at least the factorial: n! <= (n+1)!.
theorem factorial_real_lte_suc(n: Nat) {
    Real.from_rat(Rat.from_nat(n.factorial)) <= Real.from_rat(Rat.from_nat(n.suc.factorial))
} by {
    factorial_suc_real(n)
    Real.from_rat(Rat.from_nat(n.suc.factorial)) =
        Real.from_rat(Rat.from_nat(n.suc)) * Real.from_rat(Rat.from_nat(n.factorial))
    Nat.1 <= n.suc
    rat_from_nat_lte_of_nat_lte(Nat.1, n.suc)
    Rat.1 <= Rat.from_nat(n.suc)
    Real.from_rat(Rat.1) <= Real.from_rat(Rat.from_nat(n.suc))
    Real.from_rat(Rat.1) = Real.1
    Real.1 <= Real.from_rat(Rat.from_nat(n.suc))
    factorial_pos(n)
    Real.from_rat(Rat.from_nat(n.factorial)) > Real.0
    Real.from_rat(Rat.from_nat(n.factorial)) >= Real.0
    Real.1 >= Real.0
    Real.from_rat(Rat.from_nat(n.suc)) >= Real.0
    mul_le_mul_nonneg(Real.1, Real.from_rat(Rat.from_nat(n.factorial)),
        Real.from_rat(Rat.from_nat(n.suc)), Real.from_rat(Rat.from_nat(n.factorial)))
    Real.1 * Real.from_rat(Rat.from_nat(n.factorial)) <= Real.from_rat(Rat.from_nat(n.suc)) * Real.from_rat(Rat.from_nat(n.factorial))
    Real.1 * Real.from_rat(Rat.from_nat(n.factorial)) = Real.from_rat(Rat.from_nat(n.factorial))
    Real.from_rat(Rat.from_nat(n.factorial)) <= Real.from_rat(Rat.from_nat(n.suc.factorial))
}

/// The absolute value of a shifted exponential term.
/// |x^n / (n+1)!| = |x|^n / (n+1)! since the factorial is positive.
theorem exp_shift_term_abs(x: Real, n: Nat) {
    exp_shift_term(x, n).abs = x.abs.pow(n) / Real.from_rat(Rat.from_nat(n.suc.factorial))
} by {
    let denom = Real.from_rat(Rat.from_nat(n.suc.factorial))
    denom > Real.0
    denom != Real.0
    exp_shift_term(x, n).abs = (x.pow(n) / denom).abs
    (x.pow(n) / denom).abs = x.pow(n).abs / denom.abs
    x.pow(n).abs = x.abs.pow(n)
    not denom.is_negative
    denom.abs = denom
    x.pow(n).abs / denom.abs = x.abs.pow(n) / denom
    exp_shift_term(x, n).abs = x.abs.pow(n) / denom
}

/// A shifted exponential term is dominated by the corresponding exponential term.
/// |x|^n / (n+1)! <= |x|^n / n!.
theorem exp_shift_term_abs_le_exp_term_abs(x: Real, n: Nat) {
    exp_shift_term(x, n).abs <= exp_term(x, n).abs
} by {
    exp_shift_term_abs(x, n)
    exp_term_abs(x, n)
    let denom_suc = Real.from_rat(Rat.from_nat(n.suc.factorial))
    let denom = Real.from_rat(Rat.from_nat(n.factorial))
    exp_shift_term(x, n).abs = x.abs.pow(n) / denom_suc
    exp_term(x, n).abs = x.abs.pow(n) / denom
    denom > Real.0
    denom_suc > Real.0
    factorial_real_lte_suc(n)
    Real.from_rat(Rat.from_nat(n.factorial)) <= Real.from_rat(Rat.from_nat(n.suc.factorial))
    denom <= denom_suc
    real_recip_antitone_pos(denom, denom_suc)
    Real.1 / denom_suc <= Real.1 / denom
    pow_nonneg(x.abs, n)
    Real.0 <= x.abs.pow(n)
    lte_mul_nonneg_right(Real.1 / denom_suc, Real.1 / denom, x.abs.pow(n))
    (Real.1 / denom_suc) * x.abs.pow(n) <= (Real.1 / denom) * x.abs.pow(n)
    x.abs.pow(n) * (Real.1 / denom_suc) <= x.abs.pow(n) * (Real.1 / denom)
    x.abs.pow(n) / denom_suc <= x.abs.pow(n) / denom
    exp_shift_term(x, n).abs <= exp_term(x, n).abs
}

/// The shifted exponential series converges absolutely for every real x.
theorem exp_shift_term_abs_converges(x: Real) {
    absolutely_converges(exp_shift_term(x))
} by {
    forall(n: Nat) {
        abs_fn(exp_shift_term(x), n) = exp_shift_term(x, n).abs
        abs_fn(exp_term(x), n) = exp_term(x, n).abs
        exp_shift_term_abs_le_exp_term_abs(x, n)
        exp_shift_term(x, n).abs <= exp_term(x, n).abs
        abs_fn(exp_shift_term(x), n) <= abs_fn(exp_term(x), n)
    }
    seq_lte(abs_fn(exp_shift_term(x)), abs_fn(exp_term(x)))
    forall(n: Nat) {
        abs_fn_nonneg(exp_term(x), n)
        abs_fn(exp_term(x), n) >= Real.0
        Real.0 <= abs_fn(exp_term(x), n)
    }
    is_lower_bound(abs_fn(exp_term(x)), Real.0) = forall(n: Nat) {
        Real.0 <= abs_fn(exp_term(x), n)
    }
    is_lower_bound(abs_fn(exp_term(x)), Real.0)
    exp_term_abs_converges(x)
    absolutely_converges(exp_term(x))
    converges(partial(abs_fn(exp_term(x))))
    absolutely_converges_comparison(exp_shift_term(x), abs_fn(exp_term(x)))
    absolutely_converges(exp_shift_term(x))
}

/// The exponential satisfies x.exp - 1 = x * sum_{n>=0} x^n / (n+1)!.
theorem exp_sub_one_series(x: Real) {
    x.exp - Real.1 = x * limit(partial(exp_shift_term(x)))
} by {
    exp_shift_term_abs_converges(x)
    absolutely_converges(exp_shift_term(x))
    converges(partial(exp_shift_term(x)))
    converges_to(mul_seq(x, partial(exp_shift_term(x))),
        x * limit(partial(exp_shift_term(x))))
    converges_mul_seq(x, partial(exp_shift_term(x)))
    converges(mul_seq(x, partial(exp_shift_term(x))))
    convergent_converges_to_limit(mul_seq(x, partial(exp_shift_term(x))))
    converges_to(mul_seq(x, partial(exp_shift_term(x))),
        limit(mul_seq(x, partial(exp_shift_term(x)))))
    converges_to_unique(mul_seq(x, partial(exp_shift_term(x))),
        limit(mul_seq(x, partial(exp_shift_term(x)))),
        x * limit(partial(exp_shift_term(x))))
    limit(mul_seq(x, partial(exp_shift_term(x)))) = x * limit(partial(exp_shift_term(x)))

    forall(n: Nat) {
        exp_partial_shift_mul(x, n)
        x * partial(exp_shift_term(x), n) = partial(exp_term(x), n.suc) - Real.1
        mul_seq(x, partial(exp_shift_term(x)), n) = x * partial(exp_shift_term(x), n)
        tail(partial(exp_term(x)), Nat.1, n) = partial(exp_term(x), Nat.1 + n)
        Nat.1 + n = n.suc
        tail(partial(exp_term(x)), Nat.1, n) = partial(exp_term(x), n.suc)
        neg_seq(const_seq(Real.1), n) = mul_seq(-Real.1, const_seq(Real.1), n)
        mul_seq(-Real.1, const_seq(Real.1), n) = -Real.1 * const_seq(Real.1, n)
        const_seq(Real.1, n) = Real.1
        -Real.1 * Real.1 = -Real.1
        neg_seq(const_seq(Real.1), n) = -Real.1
        add_seq(tail(partial(exp_term(x)), Nat.1), neg_seq(const_seq(Real.1)), n) =
            partial(exp_term(x), n.suc) - Real.1
        mul_seq(x, partial(exp_shift_term(x)), n) =
            add_seq(tail(partial(exp_term(x)), Nat.1), neg_seq(const_seq(Real.1)), n)
    }
    mul_seq(x, partial(exp_shift_term(x))) =
        add_seq(tail(partial(exp_term(x)), Nat.1), neg_seq(const_seq(Real.1)))
    limit(mul_seq(x, partial(exp_shift_term(x)))) =
        limit(add_seq(tail(partial(exp_term(x)), Nat.1), neg_seq(const_seq(Real.1))))

    exp_term_partial_converges(x)
    converges(partial(exp_term(x)))
    tail_converges_to(partial(exp_term(x)), Nat.1)
    converges_to(tail(partial(exp_term(x)), Nat.1), limit(partial(exp_term(x))))
    converges(tail(partial(exp_term(x)), Nat.1))
    const_converges(Real.1)
    converges(const_seq(Real.1))
    neg_seq_converges(const_seq(Real.1))
    converges(neg_seq(const_seq(Real.1)))
    limit_add_seq(tail(partial(exp_term(x)), Nat.1), neg_seq(const_seq(Real.1)))
    add_seq_converges(tail(partial(exp_term(x)), Nat.1), neg_seq(const_seq(Real.1)))
    converges(add_seq(tail(partial(exp_term(x)), Nat.1), neg_seq(const_seq(Real.1))))
    convergent_converges_to_limit(add_seq(tail(partial(exp_term(x)), Nat.1), neg_seq(const_seq(Real.1))))
    converges_to(add_seq(tail(partial(exp_term(x)), Nat.1), neg_seq(const_seq(Real.1))),
        limit(add_seq(tail(partial(exp_term(x)), Nat.1), neg_seq(const_seq(Real.1)))))
    converges_to_unique(add_seq(tail(partial(exp_term(x)), Nat.1), neg_seq(const_seq(Real.1))),
        limit(add_seq(tail(partial(exp_term(x)), Nat.1), neg_seq(const_seq(Real.1)))),
        limit(tail(partial(exp_term(x)), Nat.1)) + limit(neg_seq(const_seq(Real.1))))
    limit(add_seq(tail(partial(exp_term(x)), Nat.1), neg_seq(const_seq(Real.1)))) =
        limit(tail(partial(exp_term(x)), Nat.1)) + limit(neg_seq(const_seq(Real.1)))
    convergent_converges_to_limit(tail(partial(exp_term(x)), Nat.1))
    converges_to(tail(partial(exp_term(x)), Nat.1),
        limit(tail(partial(exp_term(x)), Nat.1)))
    converges_to_unique(tail(partial(exp_term(x)), Nat.1),
        limit(tail(partial(exp_term(x)), Nat.1)), limit(partial(exp_term(x))))
    limit(tail(partial(exp_term(x)), Nat.1)) = limit(partial(exp_term(x)))
    neg_seq_converges_to(const_seq(Real.1))
    converges_to(neg_seq(const_seq(Real.1)), -limit(const_seq(Real.1)))
    const_limit(Real.1)
    limit(const_seq(Real.1)) = Real.1
    limit(neg_seq(const_seq(Real.1))) = -Real.1
    limit(add_seq(tail(partial(exp_term(x)), Nat.1), neg_seq(const_seq(Real.1)))) =
        limit(partial(exp_term(x))) + -Real.1
    limit(partial(exp_term(x))) = x.exp
    limit(add_seq(tail(partial(exp_term(x)), Nat.1), neg_seq(const_seq(Real.1)))) =
        x.exp - Real.1

    x * limit(partial(exp_shift_term(x))) = x.exp - Real.1
    x.exp - Real.1 = x * limit(partial(exp_shift_term(x)))
}

/// One half is positive.
theorem half_pos {
    Real.1 / two > Real.0
} by {
    two.is_positive
    inverse_pos(two)
    two.inverse.is_positive
    Real.1.is_positive
    mul_pos_pos(Real.1, two.inverse)
    Real.1 * two.inverse > Real.0
    Real.1 / two = Real.1 * two.inverse
    Real.1 / two > Real.0
}

/// One half is less than one.
theorem half_lt_one {
    Real.1 / two < Real.1
} by {
    two > Real.1
    Real.1 < two
    Real.1 > Real.0
    real_inverse_antitone_pos_strict(Real.1, two)
    two.inverse < Real.1.inverse
    Real.1.inverse = Real.1
    two.inverse < Real.1
    Real.1 / two = Real.1 * two.inverse
    Real.1 * two.inverse < Real.1 * Real.1
    Real.1 * Real.1 = Real.1
    Real.1 / two < Real.1
}

/// The absolute value of a partial sum is at most the sum of the absolute values.
theorem partial_abs_le_partial_abs_fn(a: Nat -> Real, n: Nat) {
    partial(a, n).abs <= partial(abs_fn(a), n)
} by {
    let items = map(n.range, a)
    partial(a, n) = sum(items)
    sum_abs_le_abs_sum(items)
    sum(items).abs <= sum(map(items, Real.abs))
    partial(a, n).abs = sum(items).abs
    partial_abs_fn_eq(a, n)
    partial(abs_fn(a), n) = sum(map(map(n.range, a), Real.abs))
    sum(map(items, Real.abs)) = sum(map(map(n.range, a), Real.abs))
    partial(a, n).abs <= partial(abs_fn(a), n)
}

/// A positive factorial of a successor index is at least two: (j+1)! >= 2.
theorem factorial_suc_real_ge_two(j: Nat) {
    j >= Nat.1 implies Real.from_rat(Rat.from_nat(j.suc.factorial)) >= two
} by {
    if j >= Nat.1 {
        factorial_suc_real(j)
        Real.from_rat(Rat.from_nat(j.suc.factorial)) =
            Real.from_rat(Rat.from_nat(j.suc)) * Real.from_rat(Rat.from_nat(j.factorial))
        j.suc >= Nat.2
        rat_from_nat_lte_of_nat_lte(Nat.2, j.suc)
        Rat.from_nat(Nat.2) <= Rat.from_nat(j.suc)
        Real.from_rat(Rat.from_nat(Nat.2)) <= Real.from_rat(Rat.from_nat(j.suc))
        from_nat_is_from_rat(Nat.2)
        Real.from_rat(Rat.from_nat(Nat.2)) = from_nat[Real](Nat.2)
        from_nat_add[Real](Nat.1, Nat.1)
        from_nat[Real](Nat.1 + Nat.1) = from_nat[Real](Nat.1) + from_nat[Real](Nat.1)
        Nat.1 + Nat.1 = Nat.2
        from_nat[Real](Nat.2) = from_nat[Real](Nat.1) + from_nat[Real](Nat.1)
        from_nat[Real](Nat.1) = Real.1
        from_nat[Real](Nat.2) = Real.1 + Real.1
        Real.1 + Real.1 = two
        Real.from_rat(Rat.from_nat(Nat.2)) = two
        two <= Real.from_rat(Rat.from_nat(j.suc))
        factorial_pos(j)
        Real.from_rat(Rat.from_nat(j.factorial)) > Real.0
        Real.from_rat(Rat.from_nat(j.factorial)) >= Real.0
        Nat.1 <= j.factorial
        rat_from_nat_lte_of_nat_lte(Nat.1, j.factorial)
        Rat.1 <= Rat.from_nat(j.factorial)
        Real.from_rat(Rat.1) <= Real.from_rat(Rat.from_nat(j.factorial))
        Real.from_rat(Rat.1) = Real.1
        Real.from_rat(Rat.from_nat(j.factorial)) >= Real.1
        Real.1 >= Real.0
        two.is_positive
        two > Real.0
        two >= Real.0
        suc_pos(j)
        Real.from_rat(Rat.from_nat(j.suc)) > Real.0
        Real.from_rat(Rat.from_nat(j.suc)) >= Real.0
        mul_le_mul_nonneg(two, Real.1, Real.from_rat(Rat.from_nat(j.suc)), Real.from_rat(Rat.from_nat(j.factorial)))
        two * Real.1 <= Real.from_rat(Rat.from_nat(j.suc)) * Real.from_rat(Rat.from_nat(j.factorial))
        two * Real.1 = two
        two <= Real.from_rat(Rat.from_nat(j.suc)) * Real.from_rat(Rat.from_nat(j.factorial))
        Real.from_rat(Rat.from_nat(j.suc.factorial)) >= two
    }
}

/// A shifted exponential term is at most |x|^j / 2 for positive indices.
theorem exp_shift_term_abs_le_half_pow(h: Real, j: Nat) {
    h.abs < Real.1 / two and j >= Nat.1 implies exp_shift_term(h, j).abs <= h.abs.pow(j) / two
} by {
    if h.abs < Real.1 / two and j >= Nat.1 {
        exp_shift_term_abs(h, j)
        exp_shift_term(h, j).abs = h.abs.pow(j) / Real.from_rat(Rat.from_nat(j.suc.factorial))
        factorial_suc_real_ge_two(j)
        Real.from_rat(Rat.from_nat(j.suc.factorial)) >= two
        two > Real.0
        Real.from_rat(Rat.from_nat(j.suc.factorial)) > Real.0
        real_recip_antitone_pos(two, Real.from_rat(Rat.from_nat(j.suc.factorial)))
        Real.1 / Real.from_rat(Rat.from_nat(j.suc.factorial)) <= Real.1 / two
        pow_nonneg(h.abs, j)
        Real.0 <= h.abs.pow(j)
        lte_mul_nonneg_right(Real.1 / Real.from_rat(Rat.from_nat(j.suc.factorial)),
            Real.1 / two, h.abs.pow(j))
(Real.1 / Real.from_rat(Rat.from_nat(j.suc.factorial))) * h.abs.pow(j) <= (Real.1 / two) * h.abs.pow(j)
h.abs.pow(j) * (Real.1 / Real.from_rat(Rat.from_nat(j.suc.factorial))) <= h.abs.pow(j) * (Real.1 / two)
        h.abs.pow(j) / Real.from_rat(Rat.from_nat(j.suc.factorial)) <= h.abs.pow(j) / two
        exp_shift_term(h, j).abs <= h.abs.pow(j) / two
    }
}

/// The geometric partial sums are bounded by 1 / (1 - |h|).
theorem geom_partial_bound(h: Real, n: Nat) {
    h.abs < Real.1 implies partial(h.abs.pow, n) <= Real.1 / (Real.1 - h.abs)
} by {
    if h.abs < Real.1 {
        abs_gte_zero(h)
        Real.0 <= h.abs
        pos_geom_indirect_upper_bound(h.abs, n)
        partial(h.abs.pow, n) * (Real.1 - h.abs) <= Real.1
        Real.1 - h.abs > Real.0
        div_le_of_mul_le(partial(h.abs.pow, n), Real.1 - h.abs, Real.1)
        partial(h.abs.pow, n) <= Real.1 / (Real.1 - h.abs)
    }
}

/// The geometric partial sums starting from the first power are bounded by |h| / (1 - |h|).
theorem geom_partial_minus_one_bound(h: Real, n: Nat) {
    h.abs < Real.1 implies partial(h.abs.pow, n.suc) - Real.1 <= h.abs / (Real.1 - h.abs)
} by {
    if h.abs < Real.1 {
        geom_partial_bound(h, n.suc)
        partial(h.abs.pow, n.suc) <= Real.1 / (Real.1 - h.abs)
        Real.1 - h.abs > Real.0
        Real.1 - h.abs != Real.0
        Real.1 + -h.abs = Real.1 - h.abs
        Real.1 - (Real.1 - h.abs) = Real.1 + -(Real.1 - h.abs)
        -(Real.1 - h.abs) = -Real.1 + h.abs
        Real.1 + -(Real.1 - h.abs) = Real.1 + (-Real.1 + h.abs)
        Real.1 + (-Real.1 + h.abs) = (Real.1 + -Real.1) + h.abs
        Real.1 + -Real.1 = Real.0
        (Real.1 + -Real.1) + h.abs = Real.0 + h.abs
        Real.0 + h.abs = h.abs
        Real.1 + (-Real.1 + h.abs) = h.abs
        Real.1 - (Real.1 - h.abs) = h.abs
        (Real.1 - (Real.1 - h.abs)) / (Real.1 - h.abs) = h.abs / (Real.1 - h.abs)
        div_add_distrib(Real.1, -(Real.1 - h.abs), Real.1 - h.abs)
        (Real.1 + -(Real.1 - h.abs)) / (Real.1 - h.abs) =
            Real.1 / (Real.1 - h.abs) + (-(Real.1 - h.abs)) / (Real.1 - h.abs)
        Real.1 + -(Real.1 - h.abs) = Real.1 - (Real.1 - h.abs)
        (Real.1 - (Real.1 - h.abs)) / (Real.1 - h.abs) =
            Real.1 / (Real.1 - h.abs) + (-(Real.1 - h.abs)) / (Real.1 - h.abs)
        (-(Real.1 - h.abs)) / (Real.1 - h.abs) = -Real.1
        Real.1 / (Real.1 - h.abs) + (-(Real.1 - h.abs)) / (Real.1 - h.abs) =
            Real.1 / (Real.1 - h.abs) - Real.1
        h.abs / (Real.1 - h.abs) = Real.1 / (Real.1 - h.abs) - Real.1
        Real.1 / (Real.1 - h.abs) - Real.1 = h.abs / (Real.1 - h.abs)
        lte_add_right(partial(h.abs.pow, n.suc), Real.1 / (Real.1 - h.abs), -Real.1)
        partial(h.abs.pow, n.suc) + -Real.1 <= Real.1 / (Real.1 - h.abs) + -Real.1
        partial(h.abs.pow, n.suc) - Real.1 <= Real.1 / (Real.1 - h.abs) - Real.1
        partial(h.abs.pow, n.suc) - Real.1 <= h.abs / (Real.1 - h.abs)
    }
}

/// The half-scaled geometric tail is bounded by |h| when |h| <= 1/2.
theorem half_geom_tail_bound(h: Real) {
    h.abs < Real.1 / two implies (Real.1 / two) * (h.abs / (Real.1 - h.abs)) <= h.abs
} by {
    if h.abs < Real.1 / two {
        lt_mul_pos_left(h.abs, Real.1 / two, two)
        two * h.abs < two * (Real.1 / two)
        two * (Real.1 / two) = Real.1
        two * h.abs < Real.1
        two * h.abs <= Real.1
        half_lt_one
        Real.1 / two < Real.1
        lt_trans(h.abs, Real.1 / two, Real.1)
        h.abs < Real.1
        Real.1 - h.abs > Real.0
        two * (Real.1 - h.abs) = two - two * h.abs
        two * h.abs <= Real.1
        -(two * h.abs) >= -Real.1
        -Real.1 <= -(two * h.abs)
        lte_add_right(-Real.1, -(two * h.abs), two)
        -Real.1 + two <= -(two * h.abs) + two
        two + -(two * h.abs) >= two + -Real.1
        two - two * h.abs >= two - Real.1
        two - Real.1 = Real.1
        two - two * h.abs >= Real.1
        two * (Real.1 - h.abs) >= Real.1
        Real.1 <= two * (Real.1 - h.abs)
        half_pos
        Real.1 / (Real.1 - h.abs) > Real.0
        lte_mul_nonneg_right(Real.1, two * (Real.1 - h.abs), Real.1 / (Real.1 - h.abs))
        Real.1 * (Real.1 / (Real.1 - h.abs)) <= (two * (Real.1 - h.abs)) * (Real.1 / (Real.1 - h.abs))
        Real.1 * (Real.1 / (Real.1 - h.abs)) = Real.1 / (Real.1 - h.abs)
        Real.1 - h.abs != Real.0
        (Real.1 - h.abs) * (Real.1 / (Real.1 - h.abs)) = Real.1
        (two * (Real.1 - h.abs)) * (Real.1 / (Real.1 - h.abs)) =
            two * ((Real.1 - h.abs) * (Real.1 / (Real.1 - h.abs)))
        (two * (Real.1 - h.abs)) * (Real.1 / (Real.1 - h.abs)) = two * Real.1
        two * Real.1 = two
        (two * (Real.1 - h.abs)) * (Real.1 / (Real.1 - h.abs)) = two
        Real.1 / (Real.1 - h.abs) <= two
        abs_gte_zero(h)
        Real.0 <= h.abs
        lte_mul_nonneg_right(Real.1 / (Real.1 - h.abs), two, h.abs)
        (Real.1 / (Real.1 - h.abs)) * h.abs <= two * h.abs
        h.abs * (Real.1 / (Real.1 - h.abs)) <= two * h.abs
        h.abs / (Real.1 - h.abs) <= two * h.abs
        half_pos
        Real.1 / two > Real.0
        not (Real.1 / two).is_negative
        lte_mul_nonneg_right(h.abs / (Real.1 - h.abs), two * h.abs, Real.1 / two)
        (h.abs / (Real.1 - h.abs)) * (Real.1 / two) <= (two * h.abs) * (Real.1 / two)
        (Real.1 / two) * (h.abs / (Real.1 - h.abs)) <= (Real.1 / two) * (two * h.abs)
        (Real.1 / two) * (two * h.abs) = h.abs
        (Real.1 / two) * (h.abs / (Real.1 - h.abs)) <= h.abs
    }
}

/// The shifted-series partial sums from index one are bounded by |h| when |h| <= 1/2.
theorem shift_partial_sum_bound(h: Real, n: Nat) {
    h.abs < Real.1 / two implies partial(tail(abs_fn(exp_shift_term(h)), Nat.1), n) <= h.abs
} by {
    if h.abs < Real.1 / two {
        forall(j: Nat) {
            tail(abs_fn(exp_shift_term(h)), Nat.1, j) =
                abs_fn(exp_shift_term(h))(Nat.1 + j)
            abs_fn(exp_shift_term(h))(Nat.1 + j) = exp_shift_term(h, Nat.1 + j).abs
            Nat.1 + j = j.suc
            abs_fn(exp_shift_term(h))(Nat.1 + j) = exp_shift_term(h, j.suc).abs
            j.suc >= Nat.1
            exp_shift_term_abs_le_half_pow(h, j.suc)
            exp_shift_term(h, j.suc).abs <= h.abs.pow(j.suc) / two
            tail(abs_fn(exp_shift_term(h)), Nat.1, j) <= h.abs.pow(j.suc) / two
            tail(h.abs.pow, Nat.1, j) = h.abs.pow(Nat.1 + j)
            h.abs.pow(Nat.1 + j) = h.abs.pow(j.suc)
            mul_seq(Real.1 / two, tail(h.abs.pow, Nat.1), j) =
                (Real.1 / two) * tail(h.abs.pow, Nat.1, j)
            (Real.1 / two) * tail(h.abs.pow, Nat.1, j) = h.abs.pow(j.suc) / two
tail(abs_fn(exp_shift_term(h)), Nat.1, j) <= mul_seq(Real.1 / two, tail(h.abs.pow, Nat.1), j)
        }
        seq_lte(tail(abs_fn(exp_shift_term(h)), Nat.1),
            mul_seq(Real.1 / two, tail(h.abs.pow, Nat.1)))
        partial_seq_lte(tail(abs_fn(exp_shift_term(h)), Nat.1),
            mul_seq(Real.1 / two, tail(h.abs.pow, Nat.1)))
        seq_lte(partial(tail(abs_fn(exp_shift_term(h)), Nat.1)),
            partial(mul_seq(Real.1 / two, tail(h.abs.pow, Nat.1))))
partial(tail(abs_fn(exp_shift_term(h)), Nat.1), n) <= partial(mul_seq(Real.1 / two, tail(h.abs.pow, Nat.1)), n)
        partial_mul_seq_comm(Real.1 / two, tail(h.abs.pow, Nat.1))
        partial(mul_seq(Real.1 / two, tail(h.abs.pow, Nat.1))) =
            mul_seq(Real.1 / two, partial(tail(h.abs.pow, Nat.1)))
        partial(mul_seq(Real.1 / two, tail(h.abs.pow, Nat.1)), n) =
            (Real.1 / two) * partial(tail(h.abs.pow, Nat.1), n)
        partial(tail(h.abs.pow, Nat.1), n) = partial(h.abs.pow, Nat.1 + n) - partial(h.abs.pow, Nat.1)
        partial(h.abs.pow, Nat.1) = h.abs.pow(Nat.0)
        h.abs.pow(Nat.0) = Real.1
        partial(h.abs.pow, Nat.1) = Real.1
        Nat.1 + n = n.suc
        partial(tail(h.abs.pow, Nat.1), n) = partial(h.abs.pow, n.suc) - Real.1
        (Real.1 / two) * partial(tail(h.abs.pow, Nat.1), n) =
            (Real.1 / two) * (partial(h.abs.pow, n.suc) - Real.1)
        partial(mul_seq(Real.1 / two, tail(h.abs.pow, Nat.1)), n) =
            (Real.1 / two) * (partial(h.abs.pow, n.suc) - Real.1)
        half_lt_one
        Real.1 / two < Real.1
        lt_trans(h.abs, Real.1 / two, Real.1)
        h.abs < Real.1
        geom_partial_minus_one_bound(h, n)
        partial(h.abs.pow, n.suc) - Real.1 <= h.abs / (Real.1 - h.abs)
        half_pos
        Real.1 / two > Real.0
        not (Real.1 / two).is_negative
        lte_mul_nonneg_right(partial(h.abs.pow, n.suc) - Real.1,
            h.abs / (Real.1 - h.abs), Real.1 / two)
        (partial(h.abs.pow, n.suc) - Real.1) * (Real.1 / two) <= (h.abs / (Real.1 - h.abs)) * (Real.1 / two)
        (Real.1 / two) * (partial(h.abs.pow, n.suc) - Real.1) <= (Real.1 / two) * (h.abs / (Real.1 - h.abs))
        half_geom_tail_bound(h)
        (Real.1 / two) * (h.abs / (Real.1 - h.abs)) <= h.abs
        lte_trans((Real.1 / two) * (partial(h.abs.pow, n.suc) - Real.1),
            (Real.1 / two) * (h.abs / (Real.1 - h.abs)), h.abs)
        (Real.1 / two) * (partial(h.abs.pow, n.suc) - Real.1) <= h.abs
        partial(tail(abs_fn(exp_shift_term(h)), Nat.1), n) <= (Real.1 / two) * (partial(h.abs.pow, n.suc) - Real.1)
        lte_trans(partial(tail(abs_fn(exp_shift_term(h)), Nat.1), n),
            (Real.1 / two) * (partial(h.abs.pow, n.suc) - Real.1), h.abs)
        partial(tail(abs_fn(exp_shift_term(h)), Nat.1), n) <= h.abs
    }
}

/// The shifted-series limit S(h) = sum_{n>=0} h^n/(n+1)! satisfies |S(h) - 1| <= |h|
/// when |h| <= 1/2.
theorem exp_shift_sum_close_one(h: Real) {
    h.abs < Real.1 / two implies
        (limit(partial(exp_shift_term(h))) - Real.1).abs <= h.abs
} by {
    if h.abs < Real.1 / two {
        exp_shift_term_abs_converges(h)
        absolutely_converges(exp_shift_term(h))
        converges(partial(exp_shift_term(h)))
        define a(n: Nat) -> Real {
            partial(exp_shift_term(h), n) - Real.1
        }
        forall(n: Nat) {
            tail(a, Nat.1, n) = a(Nat.1 + n)
            a(Nat.1 + n) = partial(exp_shift_term(h), Nat.1 + n) - Real.1
            Nat.1 + n = n.suc
            a(Nat.1 + n) = partial(exp_shift_term(h), n.suc) - Real.1
            partial(tail(exp_shift_term(h), Nat.1), n) =
                partial(exp_shift_term(h), Nat.1 + n) - partial(exp_shift_term(h), Nat.1)
            partial(exp_shift_term(h), Nat.1) = exp_shift_term(h, Nat.0)
            exp_shift_term(h, Nat.0) = h.pow(Nat.0) / Real.from_rat(Rat.from_nat(Nat.0.suc.factorial))
            h.pow(Nat.0) = Real.1
            Real.from_rat(Rat.from_nat(Nat.0.suc.factorial)) = Real.from_rat(Rat.from_nat(Nat.1.factorial))
            Nat.1.factorial = Nat.1
            Real.from_rat(Rat.from_nat(Nat.1)) = Real.1
            Real.1 / Real.1 = Real.1
            exp_shift_term(h, Nat.0) = Real.1
            partial(exp_shift_term(h), Nat.1) = Real.1
            partial(tail(exp_shift_term(h), Nat.1), n) =
                partial(exp_shift_term(h), n.suc) - Real.1
            tail(a, Nat.1, n) = partial(tail(exp_shift_term(h), Nat.1), n)
            partial_abs_le_partial_abs_fn(tail(exp_shift_term(h), Nat.1), n)
partial(tail(exp_shift_term(h), Nat.1), n).abs <= partial(abs_fn(tail(exp_shift_term(h), Nat.1)), n)
            abs_fn_tail_comm(exp_shift_term(h), Nat.1)
            abs_fn(tail(exp_shift_term(h), Nat.1)) = tail(abs_fn(exp_shift_term(h)), Nat.1)
partial(tail(exp_shift_term(h), Nat.1), n).abs <= partial(tail(abs_fn(exp_shift_term(h)), Nat.1), n)
            tail(a, Nat.1, n).abs = partial(tail(exp_shift_term(h), Nat.1), n).abs
            tail(a, Nat.1, n).abs <= partial(tail(abs_fn(exp_shift_term(h)), Nat.1), n)
            shift_partial_sum_bound(h, n)
            partial(tail(abs_fn(exp_shift_term(h)), Nat.1), n) <= h.abs
            lte_trans(tail(a, Nat.1, n).abs,
                partial(tail(abs_fn(exp_shift_term(h)), Nat.1), n), h.abs)
            tail(a, Nat.1, n).abs <= h.abs
        }
        abs_gte_zero(h)
        Real.0 <= h.abs
        const_converges(h.abs)
        converges(const_seq(h.abs))
        forall(n: Nat) {
            abs_seq(tail(a, Nat.1))(n) = tail(a, Nat.1, n).abs
            tail(a, Nat.1, n).abs <= h.abs
            const_seq(h.abs, n) = h.abs
            abs_seq(tail(a, Nat.1))(n) <= const_seq(h.abs, n)
        }
        seq_lte(abs_seq(tail(a, Nat.1)), const_seq(h.abs))
        add_seq_converges(partial(exp_shift_term(h)), neg_seq(const_seq(Real.1)))
        converges(add_seq(partial(exp_shift_term(h)), neg_seq(const_seq(Real.1))))
        forall(n: Nat) {
            add_seq(partial(exp_shift_term(h)), neg_seq(const_seq(Real.1)), n) =
                partial(exp_shift_term(h), n) + neg_seq(const_seq(Real.1), n)
            neg_seq(const_seq(Real.1), n) = mul_seq(-Real.1, const_seq(Real.1), n)
            mul_seq(-Real.1, const_seq(Real.1), n) = -Real.1 * const_seq(Real.1, n)
            const_seq(Real.1, n) = Real.1
            -Real.1 * Real.1 = -Real.1
            add_seq(partial(exp_shift_term(h)), neg_seq(const_seq(Real.1)), n) =
                partial(exp_shift_term(h), n) - Real.1
            add_seq(partial(exp_shift_term(h)), neg_seq(const_seq(Real.1)), n) = a(n)
        }
        add_seq(partial(exp_shift_term(h)), neg_seq(const_seq(Real.1))) = a
        converges(a)
        tail_converges_to(a, Nat.1)
        converges_to(tail(a, Nat.1), limit(a))
        converges(tail(a, Nat.1))
        limit_abs_seq(tail(a, Nat.1))
        converges_to(abs_seq(tail(a, Nat.1)), limit(tail(a, Nat.1)).abs)
        converges(abs_seq(tail(a, Nat.1)))
        convergent_converges_to_limit(abs_seq(tail(a, Nat.1)))
        converges_to(abs_seq(tail(a, Nat.1)), limit(abs_seq(tail(a, Nat.1))))
        converges_to_unique(abs_seq(tail(a, Nat.1)), limit(abs_seq(tail(a, Nat.1))),
            limit(tail(a, Nat.1)).abs)
        limit(abs_seq(tail(a, Nat.1))) = limit(tail(a, Nat.1)).abs
        seq_lte_preserves_limit(abs_seq(tail(a, Nat.1)), const_seq(h.abs))
        limit(abs_seq(tail(a, Nat.1))) <= limit(const_seq(h.abs))
        const_limit(h.abs)
        limit(const_seq(h.abs)) = h.abs
        limit(abs_seq(tail(a, Nat.1))) <= h.abs
        limit(tail(a, Nat.1)).abs <= h.abs
        convergent_converges_to_limit(tail(a, Nat.1))
        converges_to(tail(a, Nat.1), limit(tail(a, Nat.1)))
        converges_to_unique(tail(a, Nat.1), limit(tail(a, Nat.1)), limit(a))
        limit(tail(a, Nat.1)) = limit(a)
        limit(a).abs <= h.abs
        convergent_converges_to_limit(a)
        converges_to(a, limit(a))
        forall(n: Nat) {
            a(n) = partial(exp_shift_term(h), n) - Real.1
        }
        add_seq(partial(exp_shift_term(h)), neg_seq(const_seq(Real.1))) = a
        limit(a) = limit(add_seq(partial(exp_shift_term(h)), neg_seq(const_seq(Real.1))))
        limit_add_seq(partial(exp_shift_term(h)), neg_seq(const_seq(Real.1)))
        converges_to(add_seq(partial(exp_shift_term(h)), neg_seq(const_seq(Real.1))),
            limit(partial(exp_shift_term(h))) + limit(neg_seq(const_seq(Real.1))))
        convergent_converges_to_limit(add_seq(partial(exp_shift_term(h)), neg_seq(const_seq(Real.1))))
        converges_to(add_seq(partial(exp_shift_term(h)), neg_seq(const_seq(Real.1))),
            limit(add_seq(partial(exp_shift_term(h)), neg_seq(const_seq(Real.1)))))
        converges_to_unique(add_seq(partial(exp_shift_term(h)), neg_seq(const_seq(Real.1))),
            limit(add_seq(partial(exp_shift_term(h)), neg_seq(const_seq(Real.1)))),
            limit(partial(exp_shift_term(h))) + limit(neg_seq(const_seq(Real.1))))
        limit(add_seq(partial(exp_shift_term(h)), neg_seq(const_seq(Real.1)))) =
            limit(partial(exp_shift_term(h))) + limit(neg_seq(const_seq(Real.1)))
        neg_seq_converges_to(const_seq(Real.1))
        converges_to(neg_seq(const_seq(Real.1)), -limit(const_seq(Real.1)))
        converges(neg_seq(const_seq(Real.1)))
        convergent_converges_to_limit(neg_seq(const_seq(Real.1)))
        converges_to(neg_seq(const_seq(Real.1)), limit(neg_seq(const_seq(Real.1))))
        converges_to_unique(neg_seq(const_seq(Real.1)), limit(neg_seq(const_seq(Real.1))),
            -limit(const_seq(Real.1)))
        limit(neg_seq(const_seq(Real.1))) = -limit(const_seq(Real.1))
        const_limit(Real.1)
        limit(const_seq(Real.1)) = Real.1
        limit(neg_seq(const_seq(Real.1))) = -Real.1
        limit(add_seq(partial(exp_shift_term(h)), neg_seq(const_seq(Real.1)))) =
            limit(partial(exp_shift_term(h))) - Real.1
        limit(a) = limit(partial(exp_shift_term(h))) - Real.1
        (limit(partial(exp_shift_term(h))) - Real.1).abs <= h.abs
    }
}

/// The shifted exponential series sum S(x) = sum_{n>=0} x^n / (n+1)!.
define exp_shift_sum(x: Real) -> Real {
    limit(partial(exp_shift_term(x)))
}

/// The exponential satisfies x.exp - 1 = x * S(x).
theorem exp_sub_one_series_sum(x: Real) {
    x.exp - Real.1 = x * exp_shift_sum(x)
} by {
    exp_sub_one_series(x)
    x.exp - Real.1 = x * limit(partial(exp_shift_term(x)))
    exp_shift_sum(x) = limit(partial(exp_shift_term(x)))
    x.exp - Real.1 = x * exp_shift_sum(x)
}

/// The shifted series sum is close to one: |S(h) - 1| <= |h| when |h| <= 1/2.
theorem exp_shift_sum_close_one_sum(h: Real) {
    h.abs < Real.1 / two implies (exp_shift_sum(h) - Real.1).abs <= h.abs
} by {
    exp_shift_sum_close_one(h)
    (limit(partial(exp_shift_term(h))) - Real.1).abs <= h.abs
    exp_shift_sum(h) = limit(partial(exp_shift_term(h)))
    (exp_shift_sum(h) - Real.1).abs <= h.abs
}

/// The exponential function has derivative x0.exp at every point x0.
theorem exp_has_derivative_at(x0: Real) {
    has_derivative_at(Real.exp, x0, x0.exp)
} by {
    forall(eps: Real) {
        if eps.is_positive {
            exp_pos(x0)
            x0.exp > Real.0
            pos_imp_eq_abs(x0.exp)
            x0.exp.abs = x0.exp
            x0.exp.abs > Real.0
            exists_small_mul_variant_2(x0.exp, eps)
            let delta1: Real satisfy {
                delta1.is_positive and delta1 * x0.exp < eps
            }
            half_pos
            Real.1 / two > Real.0
            eps_smaller_than_both(Real.1 / two, delta1)
            let delta: Real satisfy {
                delta.is_positive and delta < Real.1 / two and delta < delta1
            }
            forall(x: Real) {
                if x != x0 and x.is_close(x0, delta) {
                    sub_ne_zero_of_ne(x, x0)
                    x - x0 != Real.0
                    x.is_close(x0, delta) = (x - x0).abs < delta
                    (x - x0).abs < delta
                    lt_trans((x - x0).abs, delta, Real.1 / two)
                    (x - x0).abs < Real.1 / two
                    exp_add(x0, x - x0)
                    (x0 + (x - x0)).exp = x0.exp * (x - x0).exp
                    x - x0 = x + -x0
                    x0 + (x - x0) = x0 + (x + -x0)
                    add_comm(x0, x + -x0)
                    x0 + (x + -x0) = (x + -x0) + x0
                    (x + -x0) + x0 = x + (-x0 + x0)
                    -x0 + x0 = Real.0
                    x + (-x0 + x0) = x + Real.0
                    x + Real.0 = x
                    x0 + (x - x0) = x
                    x.exp = x0.exp * (x - x0).exp
                    x.exp - x0.exp = x0.exp * (x - x0).exp - x0.exp
                    x0.exp * (x - x0).exp - x0.exp = x0.exp * ((x - x0).exp - Real.1)
                    x.exp - x0.exp = x0.exp * ((x - x0).exp - Real.1)
                    difference_quotient(Real.exp, x0, x) = (x.exp - x0.exp) / (x - x0)
                    difference_quotient(Real.exp, x0, x) =
                        (x0.exp * ((x - x0).exp - Real.1)) / (x - x0)
                    exp_sub_one_series_sum(x - x0)
                    (x - x0).exp - Real.1 = (x - x0) * exp_shift_sum(x - x0)
                    x0.exp * ((x - x0).exp - Real.1) =
                        x0.exp * ((x - x0) * exp_shift_sum(x - x0))
                    difference_quotient(Real.exp, x0, x) =
                        (x0.exp * ((x - x0) * exp_shift_sum(x - x0))) / (x - x0)
                    (x0.exp * ((x - x0) * exp_shift_sum(x - x0))) / (x - x0) =
                        x0.exp * (((x - x0) * exp_shift_sum(x - x0)) / (x - x0))
                    ((x - x0) * exp_shift_sum(x - x0)) / (x - x0) = exp_shift_sum(x - x0)
                    difference_quotient(Real.exp, x0, x) = x0.exp * exp_shift_sum(x - x0)
                    difference_quotient(Real.exp, x0, x) - x0.exp =
                        x0.exp * exp_shift_sum(x - x0) - x0.exp
                    x0.exp * exp_shift_sum(x - x0) - x0.exp =
                        x0.exp * (exp_shift_sum(x - x0) - Real.1)
                    difference_quotient(Real.exp, x0, x) - x0.exp =
                        x0.exp * (exp_shift_sum(x - x0) - Real.1)
                    exp_shift_sum_close_one_sum(x - x0)
                    (exp_shift_sum(x - x0) - Real.1).abs <= (x - x0).abs
                    abs_gte_zero(x0.exp)
                    Real.0 <= x0.exp.abs
                    lte_mul_nonneg_right((exp_shift_sum(x - x0) - Real.1).abs,
                        (x - x0).abs, x0.exp.abs)
(exp_shift_sum(x - x0) - Real.1).abs * x0.exp.abs <= (x - x0).abs * x0.exp.abs
x0.exp.abs * (exp_shift_sum(x - x0) - Real.1).abs <= x0.exp.abs * (x - x0).abs
                    mul_abs(x0.exp, exp_shift_sum(x - x0) - Real.1)
                    (x0.exp * (exp_shift_sum(x - x0) - Real.1)).abs =
                        x0.exp.abs * (exp_shift_sum(x - x0) - Real.1).abs
                    (difference_quotient(Real.exp, x0, x) - x0.exp).abs =
                        (x0.exp * (exp_shift_sum(x - x0) - Real.1)).abs
(difference_quotient(Real.exp, x0, x) - x0.exp).abs <= x0.exp.abs * (x - x0).abs
                    lt_mul_pos_left((x - x0).abs, delta, x0.exp.abs)
                    x0.exp.abs * (x - x0).abs < x0.exp.abs * delta
                    lte_lt_trans((difference_quotient(Real.exp, x0, x) - x0.exp).abs,
                        x0.exp.abs * (x - x0).abs, x0.exp.abs * delta)
                    (difference_quotient(Real.exp, x0, x) - x0.exp).abs < x0.exp.abs * delta
                    lt_mul_pos_left(delta, delta1, x0.exp.abs)
                    x0.exp.abs * delta < x0.exp.abs * delta1
                    lt_trans((difference_quotient(Real.exp, x0, x) - x0.exp).abs,
                        x0.exp.abs * delta, x0.exp.abs * delta1)
                    (difference_quotient(Real.exp, x0, x) - x0.exp).abs < x0.exp.abs * delta1
                    x0.exp.abs * delta1 = delta1 * x0.exp
                    delta1 * x0.exp < eps
                    x0.exp.abs * delta1 < eps
                    lt_trans((difference_quotient(Real.exp, x0, x) - x0.exp).abs,
                        x0.exp.abs * delta1, eps)
                    (difference_quotient(Real.exp, x0, x) - x0.exp).abs < eps
                    difference_quotient(Real.exp, x0, x).is_close(x0.exp, eps)
                }
            }
            exists(delta2: Real) {
                delta2.is_positive and forall(x: Real) {
                    x != x0 and x.is_close(x0, delta2)
                    implies difference_quotient(Real.exp, x0, x).is_close(x0.exp, eps)
                }
            }
        }
    }
    has_derivative_at(Real.exp, x0, x0.exp) = forall(eps: Real) {
        eps.is_positive implies exists(delta: Real) {
            delta.is_positive and forall(x: Real) {
                x != x0 and x.is_close(x0, delta)
                implies difference_quotient(Real.exp, x0, x).is_close(x0.exp, eps)
            }
        }
    }
    if not has_derivative_at(Real.exp, x0, x0.exp) {
        not forall(eps: Real) {
            eps.is_positive implies exists(delta: Real) {
                delta.is_positive and forall(x: Real) {
                    x != x0 and x.is_close(x0, delta)
                    implies difference_quotient(Real.exp, x0, x).is_close(x0.exp, eps)
                }
            }
        }
        let bad_eps: Real satisfy {
            bad_eps.is_positive and forall(delta: Real) {
                not (delta.is_positive and forall(x: Real) {
                    x != x0 and x.is_close(x0, delta)
                    implies difference_quotient(Real.exp, x0, x).is_close(x0.exp, bad_eps)
                })
            }
        }
        exists(delta: Real) {
            delta.is_positive and forall(x: Real) {
                x != x0 and x.is_close(x0, delta)
                implies difference_quotient(Real.exp, x0, x).is_close(x0.exp, bad_eps)
            }
        }
        false
    }
}

/// The exponential function is its own derivative.
theorem exp_is_derivative_fn {
    is_derivative_fn(Real.exp, Real.exp)
} by {
    forall(x: Real) {
        exp_has_derivative_at(x)
        has_derivative_at(Real.exp, x, x.exp)
    }
    is_derivative_fn(Real.exp, Real.exp) = forall(y: Real) {
        has_derivative_at(Real.exp, y, y.exp)
    }
    is_derivative_fn(Real.exp, Real.exp)
}

/// A real is close to another when it lies strictly between the shifted bounds.
theorem close_from_both_sides(a: Real, b: Real, eps: Real) {
    a > b - eps and a < b + eps implies a.is_close(b, eps)
} by {
    if a > b - eps and a < b + eps {
        a < b + eps
        lt_add_right(a, b + eps, -b)
        a + -b < b + eps + -b
        a - b < b + eps - b
        b + eps - b = eps
        a - b < eps
        b - eps < a
        -a < -(b - eps)
        -(b - eps) = eps - b
        -a < eps - b
        lt_add_left(-a, eps - b, b)
        b + -a < b + (eps - b)
        b + (eps - b) = eps
        b - a < eps
        if (a - b).is_negative {
            (a - b).abs = -(a - b)
            -(a - b) = b - a
            (a - b).abs = b - a
            (a - b).abs < eps
        } else {
            (a - b).abs = a - b
            (a - b).abs < eps
        }
        (a - b).abs < eps
        a.is_close(b, eps)
    }
}

/// The logarithm is continuous on the positive reals.
/// For any tolerance delta2, inputs sufficiently close to the positive x0
/// have logarithm values within delta2.
theorem log_value_close(x0: Real, delta2: Real) {
    x0 > Real.0 and delta2.is_positive implies exists(delta: Real) {
        delta.is_positive and delta < x0 and forall(x: Real) {
            x.is_close(x0, delta) implies x.log.get_or_else(Real.0).is_close(x0.log.get_or_else(Real.0), delta2)
        }
    }
} by {
    if x0 > Real.0 and delta2.is_positive {
        log_some_of_pos_exists(x0)
        let lx0: Real satisfy {
            x0.log = Option.some(lx0)
        }
        option_get_or_else_some[Real](lx0, Real.0)
        option_get_or_else(Option.some(lx0), Real.0) = lx0
        x0.log.get_or_else(Real.0) = lx0
        exp_log_or_zero(x0, lx0)
        lx0.exp = x0
        (x0.log.get_or_else(Real.0)).exp = x0
        gt_zero_imp_pos(x0)
        x0.is_positive
        delta2.is_positive
        mul_pos_pos(x0, delta2)
        (x0 * delta2).is_positive
        pos_gt_zero(x0 * delta2)
        x0 * delta2 > Real.0
        exp_gt_one_plus_x(delta2)
        delta2.exp > Real.1 + delta2
        pos_gt_zero(delta2)
        delta2 > Real.0
        lt_add_left(Real.0, delta2, Real.1)
        Real.1 + Real.0 < Real.1 + delta2
        Real.1 + Real.0 = Real.1
        Real.1 < Real.1 + delta2
        Real.1 + delta2 > Real.1
        delta2.exp > Real.1
        exp_neg(delta2)
        (-delta2).exp = Real.1 / delta2.exp
        Real.1 > Real.0
        real_inverse_antitone_pos_strict(Real.1, delta2.exp)
        delta2.exp.inverse < Real.1.inverse
        Real.1.inverse = Real.1
        delta2.exp.inverse < Real.1
        Real.1 / delta2.exp = delta2.exp.inverse
        (-delta2).exp < Real.1
        Real.1 - (-delta2).exp > Real.0
        gt_zero_imp_pos(Real.1 - (-delta2).exp)
        (Real.1 - (-delta2).exp).is_positive
        mul_pos_pos(x0, Real.1 - (-delta2).exp)
        (x0 * (Real.1 - (-delta2).exp)).is_positive
        pos_gt_zero(x0 * (Real.1 - (-delta2).exp))
        x0 * (Real.1 - (-delta2).exp) > Real.0
        eps_smaller_than_both(x0 * delta2, x0 * (Real.1 - (-delta2).exp))
        let m: Real satisfy {
            m.is_positive and m < x0 * delta2 and m < x0 * (Real.1 - (-delta2).exp)
        }
        eps_smaller_than_both(m, x0)
        let delta: Real satisfy {
            delta.is_positive and delta < m and delta < x0
        }
        lt_trans(delta, m, x0 * delta2)
        delta < x0 * delta2
        lt_trans(delta, m, x0 * (Real.1 - (-delta2).exp))
        delta < x0 * (Real.1 - (-delta2).exp)
        forall(x: Real) {
            if x.is_close(x0, delta) {
                x.is_close(x0, delta) = (x - x0).abs < delta
                (x - x0).abs < delta
                lt_trans((x - x0).abs, delta, x0)
                (x - x0).abs < x0
                abs_neg(x - x0)
                (-(x - x0)).abs = (x - x0).abs
                x0 - x = -(x - x0)
                (x0 - x).abs = (x - x0).abs
                (x0 - x).abs < x0
                lte_abs(x0 - x)
                x0 - x <= (x0 - x).abs
                lte_lt_trans(x0 - x, (x0 - x).abs, x0)
                x0 - x < x0
                lt_add_right(x0 - x, x0, -x0)
                x0 - x + -x0 < x0 + -x0
                x0 - x - x0 < x0 - x0
                x0 - x - x0 = -x
                x0 - x0 = Real.0
                -x < Real.0
                x > Real.0
                log_some_of_pos_exists(x)
                let lx: Real satisfy {
                    x.log = Option.some(lx)
                }
                option_get_or_else_some[Real](lx, Real.0)
                option_get_or_else(Option.some(lx), Real.0) = lx
                x.log.get_or_else(Real.0) = lx
                exp_log_or_zero(x, lx)
                lx.exp = x
                (x.log.get_or_else(Real.0)).exp = x
                if not (x0.log.get_or_else(Real.0) - delta2 < x.log.get_or_else(Real.0) and x.log.get_or_else(Real.0) < x0.log.get_or_else(Real.0) + delta2) {
                    not (x0.log.get_or_else(Real.0) - delta2 < x.log.get_or_else(Real.0) and x.log.get_or_else(Real.0) < x0.log.get_or_else(Real.0) + delta2)
                    if not (x.log.get_or_else(Real.0) < x0.log.get_or_else(Real.0) + delta2) {
                        not_lt_imp_gte(x.log.get_or_else(Real.0), x0.log.get_or_else(Real.0) + delta2)
                        x.log.get_or_else(Real.0) >= x0.log.get_or_else(Real.0) + delta2
                        if x.log.get_or_else(Real.0) = x0.log.get_or_else(Real.0) + delta2 {
                            (x.log.get_or_else(Real.0)).exp = (x0.log.get_or_else(Real.0) + delta2).exp
                            (x.log.get_or_else(Real.0)).exp >= (x0.log.get_or_else(Real.0) + delta2).exp
                        } else {
                            x.log.get_or_else(Real.0) > x0.log.get_or_else(Real.0) + delta2
                            exp_increasing(x0.log.get_or_else(Real.0) + delta2, x.log.get_or_else(Real.0))
                            (x0.log.get_or_else(Real.0) + delta2).exp < (x.log.get_or_else(Real.0)).exp
                            (x.log.get_or_else(Real.0)).exp >= (x0.log.get_or_else(Real.0) + delta2).exp
                        }
                        (x.log.get_or_else(Real.0)).exp >= (x0.log.get_or_else(Real.0) + delta2).exp
                        exp_add(x0.log.get_or_else(Real.0), delta2)
                        (x0.log.get_or_else(Real.0) + delta2).exp = (x0.log.get_or_else(Real.0)).exp * delta2.exp
                        (x0.log.get_or_else(Real.0)).exp = x0
                        (x0.log.get_or_else(Real.0) + delta2).exp = x0 * delta2.exp
                        (x.log.get_or_else(Real.0)).exp = x
                        x >= x0 * delta2.exp
                        lte_add_right(x0 * delta2.exp, x, -x0)
                        x0 * delta2.exp + -x0 <= x + -x0
                        x0 * delta2.exp - x0 <= x - x0
                        x - x0 >= x0 * delta2.exp - x0
                        x0 * delta2.exp - x0 = x0 * (delta2.exp - Real.1)
                        x - x0 >= x0 * (delta2.exp - Real.1)
                        delta2.exp > Real.1 + delta2
                        Real.1 + delta2 < delta2.exp
                        lt_add_right(Real.1 + delta2, delta2.exp, -Real.1)
                        Real.1 + delta2 + -Real.1 < delta2.exp + -Real.1
                        Real.1 + delta2 + -Real.1 = delta2 + Real.1 + -Real.1
                        delta2 + Real.1 + -Real.1 = delta2 + (Real.1 + -Real.1)
                        Real.1 + -Real.1 = Real.0
                        delta2 + (Real.1 + -Real.1) = delta2 + Real.0
                        delta2 + Real.0 = delta2
                        Real.1 + delta2 + -Real.1 = delta2
                        delta2 < delta2.exp - Real.1
                        delta2.exp - Real.1 > delta2
                        lt_mul_pos_left(delta2, delta2.exp - Real.1, x0)
                        x0 * delta2 < x0 * (delta2.exp - Real.1)
                        x0 * (delta2.exp - Real.1) > x0 * delta2
                        x0 * delta2 < x0 * (delta2.exp - Real.1)
                        lt_of_lt_of_lte(x0 * delta2, x0 * (delta2.exp - Real.1), x - x0)
                        x0 * delta2 < x - x0
                        x - x0 > x0 * delta2
                        x0 * delta2 > Real.0
                        lt_trans(Real.0, x0 * delta2, x - x0)
                        x - x0 > Real.0
                        pos_imp_eq_abs(x - x0)
                        (x - x0).abs = x - x0
                        (x - x0).abs > x0 * delta2
                        lt_trans((x - x0).abs, delta, x0 * delta2)
                        (x - x0).abs < x0 * delta2
                        false
                    } else {
                        x.log.get_or_else(Real.0) < x0.log.get_or_else(Real.0) + delta2
                        not (x0.log.get_or_else(Real.0) - delta2 < x.log.get_or_else(Real.0))
                        not_lt_imp_gte(x0.log.get_or_else(Real.0) - delta2, x.log.get_or_else(Real.0))
                        x0.log.get_or_else(Real.0) - delta2 >= x.log.get_or_else(Real.0)
                        x.log.get_or_else(Real.0) <= x0.log.get_or_else(Real.0) - delta2
                        if x.log.get_or_else(Real.0) = x0.log.get_or_else(Real.0) - delta2 {
                            (x.log.get_or_else(Real.0)).exp = (x0.log.get_or_else(Real.0) - delta2).exp
                            (x.log.get_or_else(Real.0)).exp <= (x0.log.get_or_else(Real.0) - delta2).exp
                        } else {
                            x.log.get_or_else(Real.0) < x0.log.get_or_else(Real.0) - delta2
                            exp_increasing(x.log.get_or_else(Real.0), x0.log.get_or_else(Real.0) - delta2)
                            (x.log.get_or_else(Real.0)).exp < (x0.log.get_or_else(Real.0) - delta2).exp
                            (x.log.get_or_else(Real.0)).exp <= (x0.log.get_or_else(Real.0) - delta2).exp
                        }
                        (x.log.get_or_else(Real.0)).exp <= (x0.log.get_or_else(Real.0) - delta2).exp
                        exp_add(x0.log.get_or_else(Real.0), -delta2)
                        (x0.log.get_or_else(Real.0) + -delta2).exp = (x0.log.get_or_else(Real.0)).exp * (-delta2).exp
                        x0.log.get_or_else(Real.0) + -delta2 = x0.log.get_or_else(Real.0) - delta2
                        (x0.log.get_or_else(Real.0) - delta2).exp = (x0.log.get_or_else(Real.0)).exp * (-delta2).exp
                        (x0.log.get_or_else(Real.0)).exp = x0
                        (x0.log.get_or_else(Real.0) - delta2).exp = x0 * (-delta2).exp
                        (x.log.get_or_else(Real.0)).exp = x
                        x <= x0 * (-delta2).exp
                        lte_add_right(x, x0 * (-delta2).exp, -(x0 * (-delta2).exp))
                        x + -(x0 * (-delta2).exp) <= x0 * (-delta2).exp + -(x0 * (-delta2).exp)
                        x - x0 * (-delta2).exp <= Real.0
                        -(x - x0 * (-delta2).exp) >= Real.0
                        -(x - x0 * (-delta2).exp) = x0 * (-delta2).exp - x
                        x0 * (-delta2).exp - x >= Real.0
                        Real.0 <= x0 * (-delta2).exp - x
                        lte_add_right(Real.0, x0 * (-delta2).exp - x, x0 - x0 * (-delta2).exp)
                        Real.0 + (x0 - x0 * (-delta2).exp) <= (x0 * (-delta2).exp - x) + (x0 - x0 * (-delta2).exp)
                        (x0 - x0 * (-delta2).exp) + (x0 * (-delta2).exp - x) >= x0 - x0 * (-delta2).exp
                        (x0 - x0 * (-delta2).exp) + (x0 * (-delta2).exp - x) = x0 - x
                        x0 - x >= x0 - x0 * (-delta2).exp
                        x0 - x0 * (-delta2).exp = x0 * (Real.1 - (-delta2).exp)
                        x0 - x >= x0 * (Real.1 - (-delta2).exp)
                        lt_mul_pos_left((-delta2).exp, Real.1, x0)
                        x0 * (-delta2).exp < x0 * Real.1
                        x0 * Real.1 = x0
                        x0 * (-delta2).exp < x0
                        lt_of_lte_of_lt(x, x0 * (-delta2).exp, x0)
                        x < x0
                        x0 - x > Real.0
                        pos_imp_eq_abs(x0 - x)
                        (x0 - x).abs = x0 - x
                        (x0 - x).abs >= x0 * (Real.1 - (-delta2).exp)
                        (x0 - x).abs = (x - x0).abs
                        (x - x0).abs >= x0 * (Real.1 - (-delta2).exp)
                        lt_trans((x - x0).abs, delta, x0 * (Real.1 - (-delta2).exp))
                        (x - x0).abs < x0 * (Real.1 - (-delta2).exp)
                        false
                    }
                }
                close_from_both_sides(x.log.get_or_else(Real.0), x0.log.get_or_else(Real.0), delta2)
                x.log.get_or_else(Real.0).is_close(x0.log.get_or_else(Real.0), delta2)
            }
        }
        exists(delta2b: Real) {
            delta2b.is_positive and delta2b < x0 and forall(x: Real) {
                x.is_close(x0, delta2b) implies x.log.get_or_else(Real.0).is_close(x0.log.get_or_else(Real.0), delta2)
            }
        }
    }
}

/// The logarithm has derivative 1/x0 at every positive point x0.
theorem log_has_derivative_at_pos(x0: Real) {
    x0 > Real.0 implies has_derivative_at(log_value, x0, Real.1 / x0)
} by {
    if x0 > Real.0 {
        x0 != Real.0
        log_some_of_pos_exists(x0)
        let lx0: Real satisfy {
            x0.log = Option.some(lx0)
        }
        option_get_or_else_some[Real](lx0, Real.0)
        option_get_or_else(Option.some(lx0), Real.0) = lx0
        x0.log.get_or_else(Real.0) = lx0
        exp_log_or_zero(x0, lx0)
        lx0.exp = x0
        (x0.log.get_or_else(Real.0)).exp = x0
        forall(eps: Real) {
            if eps.is_positive {
                continuous_at_reciprocal_real(x0)
                continuous_at(reciprocal_real, x0)
                continuous_at(reciprocal_real, x0) = forall(eps2: Real) {
                    eps2.is_positive implies exists(delta2: Real) {
                        delta2.is_positive and continuous_condition(reciprocal_real, x0, delta2, eps2)
                    }
                }
                exists(delta_r: Real) {
                    delta_r.is_positive and continuous_condition(reciprocal_real, x0, delta_r, eps)
                }
                let delta_r: Real satisfy {
                    delta_r.is_positive and continuous_condition(reciprocal_real, x0, delta_r, eps)
                }
                exp_has_derivative_at(x0.log.get_or_else(Real.0))
                has_derivative_at(Real.exp, x0.log.get_or_else(Real.0), (x0.log.get_or_else(Real.0)).exp)
                (x0.log.get_or_else(Real.0)).exp = x0
                has_derivative_at(Real.exp, x0.log.get_or_else(Real.0), x0)
                has_derivative_at(Real.exp, x0.log.get_or_else(Real.0), x0) = forall(eps3: Real) {
                    eps3.is_positive implies exists(delta3: Real) {
                        delta3.is_positive and forall(y: Real) {
                            y != x0.log.get_or_else(Real.0) and y.is_close(x0.log.get_or_else(Real.0), delta3)
                            implies difference_quotient(Real.exp, x0.log.get_or_else(Real.0), y).is_close(x0, eps3)
                        }
                    }
                }
                delta_r.is_positive
                exists(delta2: Real) {
                    delta2.is_positive and forall(y: Real) {
                        y != x0.log.get_or_else(Real.0) and y.is_close(x0.log.get_or_else(Real.0), delta2)
                        implies difference_quotient(Real.exp, x0.log.get_or_else(Real.0), y).is_close(x0, delta_r)
                    }
                }
                let delta2: Real satisfy {
                    delta2.is_positive and forall(y: Real) {
                        y != x0.log.get_or_else(Real.0) and y.is_close(x0.log.get_or_else(Real.0), delta2)
                        implies difference_quotient(Real.exp, x0.log.get_or_else(Real.0), y).is_close(x0, delta_r)
                    }
                }
                log_value_close(x0, delta2)
                let delta3: Real satisfy {
                    delta3.is_positive and delta3 < x0 and forall(x: Real) {
                        x.is_close(x0, delta3) implies x.log.get_or_else(Real.0).is_close(x0.log.get_or_else(Real.0), delta2)
                    }
                }
                forall(x: Real) {
                    if x != x0 and x.is_close(x0, delta3) {
                        x.is_close(x0, delta3) = (x - x0).abs < delta3
                        (x - x0).abs < delta3
                        lt_trans((x - x0).abs, delta3, x0)
                        (x - x0).abs < x0
                        abs_neg(x - x0)
                        (-(x - x0)).abs = (x - x0).abs
                        x0 - x = -(x - x0)
                        (x0 - x).abs = (x - x0).abs
                        (x0 - x).abs < x0
                        lte_abs(x0 - x)
                        x0 - x <= (x0 - x).abs
                        lte_lt_trans(x0 - x, (x0 - x).abs, x0)
                        x0 - x < x0
                        lt_add_right(x0 - x, x0, -x0)
                        x0 - x + -x0 < x0 + -x0
                        x0 - x - x0 < x0 - x0
                        x0 - x - x0 = -x
                        x0 - x0 = Real.0
                        -x < Real.0
                        x > Real.0
                        log_some_of_pos_exists(x)
                        let lx: Real satisfy {
                            x.log = Option.some(lx)
                        }
                        option_get_or_else_some[Real](lx, Real.0)
                        option_get_or_else(Option.some(lx), Real.0) = lx
                        x.log.get_or_else(Real.0) = lx
                        exp_log_or_zero(x, lx)
                        lx.exp = x
                        (x.log.get_or_else(Real.0)).exp = x
                        if x.log.get_or_else(Real.0) = x0.log.get_or_else(Real.0) {
                            (x.log.get_or_else(Real.0)).exp = (x0.log.get_or_else(Real.0)).exp
                            (x.log.get_or_else(Real.0)).exp = x
                            (x0.log.get_or_else(Real.0)).exp = x0
                            x = x0
                            false
                        }
                        x.log.get_or_else(Real.0) != x0.log.get_or_else(Real.0)
                        x.log.get_or_else(Real.0).is_close(x0.log.get_or_else(Real.0), delta2)
                        x.log.get_or_else(Real.0).is_close(x0.log.get_or_else(Real.0), delta2) = (x.log.get_or_else(Real.0) - x0.log.get_or_else(Real.0)).abs < delta2
                        (x.log.get_or_else(Real.0) - x0.log.get_or_else(Real.0)).abs < delta2
                        difference_quotient(Real.exp, x0.log.get_or_else(Real.0), x.log.get_or_else(Real.0)).is_close(x0, delta_r)
                        (difference_quotient(Real.exp, x0.log.get_or_else(Real.0), x.log.get_or_else(Real.0)) - x0).abs < delta_r
                        continuous_condition(reciprocal_real, x0, delta_r, eps) = forall(x1: Real) {
                            x1.is_close(x0, delta_r) implies reciprocal_real(x1).is_close(reciprocal_real(x0), eps)
                        }
                        reciprocal_real(difference_quotient(Real.exp, x0.log.get_or_else(Real.0), x.log.get_or_else(Real.0))).is_close(reciprocal_real(x0), eps)
                        difference_quotient(log_value, x0, x) = (x.log.get_or_else(Real.0) - x0.log.get_or_else(Real.0)) / (x - x0)
                        sub_ne_zero_of_ne(x, x0)
                        x - x0 != Real.0
                        (x.log.get_or_else(Real.0)).exp - (x0.log.get_or_else(Real.0)).exp = x - x0
                        difference_quotient(Real.exp, x0.log.get_or_else(Real.0), x.log.get_or_else(Real.0)) = ((x.log.get_or_else(Real.0)).exp - (x0.log.get_or_else(Real.0)).exp) / (x.log.get_or_else(Real.0) - x0.log.get_or_else(Real.0))
                        difference_quotient(Real.exp, x0.log.get_or_else(Real.0), x.log.get_or_else(Real.0)) = (x - x0) / (x.log.get_or_else(Real.0) - x0.log.get_or_else(Real.0))
                        inverse_div(x - x0, x.log.get_or_else(Real.0) - x0.log.get_or_else(Real.0))
                        ((x - x0) / (x.log.get_or_else(Real.0) - x0.log.get_or_else(Real.0))).inverse = (x.log.get_or_else(Real.0) - x0.log.get_or_else(Real.0)) / (x - x0)
                        (difference_quotient(Real.exp, x0.log.get_or_else(Real.0), x.log.get_or_else(Real.0))).inverse = (x.log.get_or_else(Real.0) - x0.log.get_or_else(Real.0)) / (x - x0)
                        reciprocal_real(difference_quotient(Real.exp, x0.log.get_or_else(Real.0), x.log.get_or_else(Real.0))) = (difference_quotient(Real.exp, x0.log.get_or_else(Real.0), x.log.get_or_else(Real.0))).inverse
                        difference_quotient(log_value, x0, x) = reciprocal_real(difference_quotient(Real.exp, x0.log.get_or_else(Real.0), x.log.get_or_else(Real.0)))
                        difference_quotient(log_value, x0, x).is_close(reciprocal_real(x0), eps)
                        reciprocal_real_apply(x0)
                        reciprocal_real(x0) = x0.inverse
                        Real.1 / x0 = Real.1 * x0.inverse
                        Real.1 * x0.inverse = x0.inverse
                        reciprocal_real(x0) = Real.1 / x0
                        difference_quotient(log_value, x0, x).is_close(Real.1 / x0, eps)
                    }
                }
                exists(delta4: Real) {
                    delta4.is_positive and forall(x: Real) {
                        x != x0 and x.is_close(x0, delta4)
                        implies difference_quotient(log_value, x0, x).is_close(Real.1 / x0, eps)
                    }
                }
            }
        }
        has_derivative_at(log_value, x0, Real.1 / x0) = forall(eps: Real) {
            eps.is_positive implies exists(delta: Real) {
                delta.is_positive and forall(x: Real) {
                    x != x0 and x.is_close(x0, delta)
                    implies difference_quotient(log_value, x0, x).is_close(Real.1 / x0, eps)
                }
            }
        }
        if not has_derivative_at(log_value, x0, Real.1 / x0) {
            not forall(eps: Real) {
                eps.is_positive implies exists(delta: Real) {
                    delta.is_positive and forall(x: Real) {
                        x != x0 and x.is_close(x0, delta)
                        implies difference_quotient(log_value, x0, x).is_close(Real.1 / x0, eps)
                    }
                }
            }
            let bad_eps: Real satisfy {
                bad_eps.is_positive and forall(delta: Real) {
                    not (delta.is_positive and forall(x: Real) {
                        x != x0 and x.is_close(x0, delta)
                        implies difference_quotient(log_value, x0, x).is_close(Real.1 / x0, bad_eps)
                    })
                }
            }
            exists(delta: Real) {
                delta.is_positive and forall(x: Real) {
                    x != x0 and x.is_close(x0, delta)
                    implies difference_quotient(log_value, x0, x).is_close(Real.1 / x0, bad_eps)
                }
            }
            false
        }
        has_derivative_at(log_value, x0, Real.1 / x0)
    }
}

/// The logarithm is differentiable with reciprocal derivative on the positive reals.
theorem log_is_derivative_fn_on_positives {
    forall(x: Real) {
        x > Real.0 implies has_derivative_at(log_value, x, Real.1 / x)
    }
} by {
    forall(x: Real) {
        log_has_derivative_at_pos(x)
        x > Real.0 implies has_derivative_at(log_value, x, Real.1 / x)
    }
}
