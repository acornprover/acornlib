/// Deep properties of real exponentiation.
///
/// This file extends `log.ac` and `log_exp_foundations.ac` with the real-power
/// algebra: powers of products and quotients of bases, powers of powers,
/// powers of reciprocals and negated exponents, and strict monotonicity
/// bounds for real powers.

from order import lt_trans, lt_of_lt_of_lte
from nat import Nat, from_nat
from real.log import Real, log_some_of_pos_exists, log_one, exp_log_or_zero, exp_neg, exp_injective, log_mul, log_rpow, rpow_add, rpow_mul, rpow_nat, rpow_pos
from real.log_exp_foundations import log_value_one, log_value_e, log_value_exp, log_value_mul, log_value_div, log_value_recip, log_value_rpow, log_rpow_val, exp_sub
from real.exp import exp_pos, exp_zero, exp_add, exp_increasing
from real.log_inequalities import log_pos_of_gt_one, log_neg_of_pos_lt_one
from real.real_ring import mul_neg_left, mul_neg_right, mul_pos_pos, lt_mul_pos_left
from real.real_base import gt_zero_imp_pos
from real.harmonic import real_one_div_pos

numerals Real

// =====================================================================
// Powers of products and quotients of bases
// =====================================================================

/// A power of a product of bases is the exponential of the scaled sum of
/// logarithms.
theorem rpow_prod_base(x: Real, y: Real, a: Real) {
    x > Real.0 and y > Real.0 implies
        (x * y).rpow(a) = Option.some((a * (x.log.get_or_else(Real.0) + y.log.get_or_else(Real.0))).exp)
} by {
    if x > Real.0 and y > Real.0 {
        x.is_positive
        y.is_positive
        (x * y).is_positive
        x * y > Real.0
        (x * y).rpow(a) = Option.some((a * (x * y).log.get_or_else(Real.0)).exp)
        log_value_mul(x, y)
        (x * y).log.get_or_else(Real.0) = x.log.get_or_else(Real.0) + y.log.get_or_else(Real.0)
        a * (x * y).log.get_or_else(Real.0) = a * (x.log.get_or_else(Real.0) + y.log.get_or_else(Real.0))
        (a * (x * y).log.get_or_else(Real.0)).exp = (a * (x.log.get_or_else(Real.0) + y.log.get_or_else(Real.0))).exp
        (x * y).rpow(a) = Option.some((a * (x.log.get_or_else(Real.0) + y.log.get_or_else(Real.0))).exp)
    }
}

/// The power of a product is the product of the powers (value form).
theorem rpow_prod_base_val(x: Real, y: Real, a: Real, u: Real, v: Real) {
    x > Real.0 and y > Real.0 and x.rpow(a) = Option.some(u) and y.rpow(a) = Option.some(v)
    implies (x * y).rpow(a) = Option.some(u * v)
} by {
    if x > Real.0 and y > Real.0 and x.rpow(a) = Option.some(u) and y.rpow(a) = Option.some(v) {
        rpow_prod_base(x, y, a)
        (x * y).rpow(a) = Option.some((a * (x.log.get_or_else(Real.0) + y.log.get_or_else(Real.0))).exp)
        x.rpow(a) = Option.some((a * x.log.get_or_else(Real.0)).exp)
        Option.some(u) = Option.some((a * x.log.get_or_else(Real.0)).exp)
        some_injective[Real](u, (a * x.log.get_or_else(Real.0)).exp)
        u = (a * x.log.get_or_else(Real.0)).exp
        y.rpow(a) = Option.some((a * y.log.get_or_else(Real.0)).exp)
        Option.some(v) = Option.some((a * y.log.get_or_else(Real.0)).exp)
        some_injective[Real](v, (a * y.log.get_or_else(Real.0)).exp)
        v = (a * y.log.get_or_else(Real.0)).exp
        a * (x.log.get_or_else(Real.0) + y.log.get_or_else(Real.0)) = a * x.log.get_or_else(Real.0) + a * y.log.get_or_else(Real.0)
        exp_add(a * x.log.get_or_else(Real.0), a * y.log.get_or_else(Real.0))
        (a * x.log.get_or_else(Real.0) + a * y.log.get_or_else(Real.0)).exp = (a * x.log.get_or_else(Real.0)).exp * (a * y.log.get_or_else(Real.0)).exp
        (a * (x.log.get_or_else(Real.0) + y.log.get_or_else(Real.0))).exp = (a * x.log.get_or_else(Real.0)).exp * (a * y.log.get_or_else(Real.0)).exp
        (a * (x.log.get_or_else(Real.0) + y.log.get_or_else(Real.0))).exp = u * v
        (x * y).rpow(a) = Option.some(u * v)
    }
}

/// A power of a quotient of bases is the exponential of the scaled difference
/// of logarithms.
theorem rpow_quot_base(x: Real, y: Real, a: Real) {
    x > Real.0 and y > Real.0 implies
        (x / y).rpow(a) = Option.some((a * (x.log.get_or_else(Real.0) - y.log.get_or_else(Real.0))).exp)
} by {
    if x > Real.0 and y > Real.0 {
        x / y > Real.0
        (x / y).rpow(a) = Option.some((a * (x / y).log.get_or_else(Real.0)).exp)
        log_value_div(x, y)
        (x / y).log.get_or_else(Real.0) = x.log.get_or_else(Real.0) - y.log.get_or_else(Real.0)
        a * (x / y).log.get_or_else(Real.0) = a * (x.log.get_or_else(Real.0) - y.log.get_or_else(Real.0))
        (a * (x / y).log.get_or_else(Real.0)).exp = (a * (x.log.get_or_else(Real.0) - y.log.get_or_else(Real.0))).exp
        (x / y).rpow(a) = Option.some((a * (x.log.get_or_else(Real.0) - y.log.get_or_else(Real.0))).exp)
    }
}

/// The power of a quotient is the quotient of the powers (value form).
theorem rpow_quot_base_val(x: Real, y: Real, a: Real, u: Real, v: Real) {
    x > Real.0 and y > Real.0 and x.rpow(a) = Option.some(u) and y.rpow(a) = Option.some(v)
    implies (x / y).rpow(a) = Option.some(u / v)
} by {
    if x > Real.0 and y > Real.0 and x.rpow(a) = Option.some(u) and y.rpow(a) = Option.some(v) {
        rpow_quot_base(x, y, a)
        (x / y).rpow(a) = Option.some((a * (x.log.get_or_else(Real.0) - y.log.get_or_else(Real.0))).exp)
        x.rpow(a) = Option.some((a * x.log.get_or_else(Real.0)).exp)
        Option.some(u) = Option.some((a * x.log.get_or_else(Real.0)).exp)
        some_injective[Real](u, (a * x.log.get_or_else(Real.0)).exp)
        u = (a * x.log.get_or_else(Real.0)).exp
        y.rpow(a) = Option.some((a * y.log.get_or_else(Real.0)).exp)
        Option.some(v) = Option.some((a * y.log.get_or_else(Real.0)).exp)
        some_injective[Real](v, (a * y.log.get_or_else(Real.0)).exp)
        v = (a * y.log.get_or_else(Real.0)).exp
        a * (x.log.get_or_else(Real.0) - y.log.get_or_else(Real.0)) = a * x.log.get_or_else(Real.0) - a * y.log.get_or_else(Real.0)
        exp_sub(a * x.log.get_or_else(Real.0), a * y.log.get_or_else(Real.0))
        (a * x.log.get_or_else(Real.0) - a * y.log.get_or_else(Real.0)).exp = (a * x.log.get_or_else(Real.0)).exp / (a * y.log.get_or_else(Real.0)).exp
        (a * (x.log.get_or_else(Real.0) - y.log.get_or_else(Real.0))).exp = (a * x.log.get_or_else(Real.0)).exp / (a * y.log.get_or_else(Real.0)).exp
        (a * (x.log.get_or_else(Real.0) - y.log.get_or_else(Real.0))).exp = u / v
        (x / y).rpow(a) = Option.some(u / v)
    }
}

/// A power of a reciprocal base is the reciprocal power (value form).
theorem rpow_inv_base(x: Real, a: Real, u: Real) {
    x > Real.0 and x.rpow(a) = Option.some(u)
    implies (Real.1 / x).rpow(a) = Option.some(Real.1 / u)
} by {
    if x > Real.0 and x.rpow(a) = Option.some(u) {
        x.rpow(a) = Option.some((a * x.log.get_or_else(Real.0)).exp)
        Option.some(u) = Option.some((a * x.log.get_or_else(Real.0)).exp)
        some_injective[Real](u, (a * x.log.get_or_else(Real.0)).exp)
        u = (a * x.log.get_or_else(Real.0)).exp
        real_one_div_pos(x)
        Real.1 / x > Real.0
        (Real.1 / x).rpow(a) = Option.some((a * (Real.1 / x).log.get_or_else(Real.0)).exp)
        log_value_recip(x)
        (Real.1 / x).log.get_or_else(Real.0) = -x.log.get_or_else(Real.0)
        a * (Real.1 / x).log.get_or_else(Real.0) = a * (-x.log.get_or_else(Real.0))
        mul_neg_right(a, x.log.get_or_else(Real.0))
        a * (-x.log.get_or_else(Real.0)) = -(a * x.log.get_or_else(Real.0))
        (a * (Real.1 / x).log.get_or_else(Real.0)).exp = (-(a * x.log.get_or_else(Real.0))).exp
        exp_neg(a * x.log.get_or_else(Real.0))
        (-(a * x.log.get_or_else(Real.0))).exp = Real.1 / (a * x.log.get_or_else(Real.0)).exp
        (a * (Real.1 / x).log.get_or_else(Real.0)).exp = Real.1 / (a * x.log.get_or_else(Real.0)).exp
        (a * (Real.1 / x).log.get_or_else(Real.0)).exp = Real.1 / u
        (Real.1 / x).rpow(a) = Option.some(Real.1 / u)
    }
}

// =====================================================================
// Powers of powers and exponent algebra
// =====================================================================

/// A power of a power is the power of the product of exponents.
theorem rpow_rpow(x: Real, a: Real, b: Real, u: Real, v: Real) {
    x > Real.0 and x.rpow(a) = Option.some(u) and u.rpow(b) = Option.some(v)
    implies x.rpow(a * b) = Option.some(v)
} by {
    if x > Real.0 and x.rpow(a) = Option.some(u) and u.rpow(b) = Option.some(v) {
        x.rpow(a) = Option.some((a * x.log.get_or_else(Real.0)).exp)
        Option.some(u) = Option.some((a * x.log.get_or_else(Real.0)).exp)
        some_injective[Real](u, (a * x.log.get_or_else(Real.0)).exp)
        u = (a * x.log.get_or_else(Real.0)).exp
        exp_pos(a * x.log.get_or_else(Real.0))
        (a * x.log.get_or_else(Real.0)).exp > Real.0
        u > Real.0
        log_value_rpow(x, a, u)
        u.log.get_or_else(Real.0) = a * x.log.get_or_else(Real.0)
        u.rpow(b) = Option.some((b * u.log.get_or_else(Real.0)).exp)
        b * u.log.get_or_else(Real.0) = b * (a * x.log.get_or_else(Real.0))
        (b * u.log.get_or_else(Real.0)).exp = (b * (a * x.log.get_or_else(Real.0))).exp
        Option.some(v) = Option.some((b * u.log.get_or_else(Real.0)).exp)
        some_injective[Real](v, (b * u.log.get_or_else(Real.0)).exp)
        v = (b * u.log.get_or_else(Real.0)).exp
        v = (b * (a * x.log.get_or_else(Real.0))).exp
        rpow_mul(x, a, b)
        x.rpow(a * b) = Option.some((b * (a * x.log.get_or_else(Real.0))).exp)
        b * (a * x.log.get_or_else(Real.0)) = (a * b) * x.log.get_or_else(Real.0)
        (b * (a * x.log.get_or_else(Real.0))).exp = ((a * b) * x.log.get_or_else(Real.0)).exp
        v = ((a * b) * x.log.get_or_else(Real.0)).exp
        x.rpow(a * b) = Option.some(v)
    }
}

/// The product of two powers of one base is the power of the sum (value form).
theorem rpow_mul_val(x: Real, a: Real, b: Real, u: Real, v: Real, w: Real) {
    x > Real.0 and x.rpow(a) = Option.some(u) and x.rpow(b) = Option.some(v) and
    x.rpow(a + b) = Option.some(w)
    implies w = u * v
} by {
    if x > Real.0 and x.rpow(a) = Option.some(u) and x.rpow(b) = Option.some(v) and
        x.rpow(a + b) = Option.some(w) {
        rpow_add(x, a, b)
        x.rpow(a + b) = Option.some((a * x.log.get_or_else(Real.0)).exp * (b * x.log.get_or_else(Real.0)).exp)
        Option.some(w) = Option.some((a * x.log.get_or_else(Real.0)).exp * (b * x.log.get_or_else(Real.0)).exp)
        some_injective[Real](w, (a * x.log.get_or_else(Real.0)).exp * (b * x.log.get_or_else(Real.0)).exp)
        w = (a * x.log.get_or_else(Real.0)).exp * (b * x.log.get_or_else(Real.0)).exp
        x.rpow(a) = Option.some((a * x.log.get_or_else(Real.0)).exp)
        Option.some(u) = Option.some((a * x.log.get_or_else(Real.0)).exp)
        some_injective[Real](u, (a * x.log.get_or_else(Real.0)).exp)
        u = (a * x.log.get_or_else(Real.0)).exp
        x.rpow(b) = Option.some((b * x.log.get_or_else(Real.0)).exp)
        Option.some(v) = Option.some((b * x.log.get_or_else(Real.0)).exp)
        some_injective[Real](v, (b * x.log.get_or_else(Real.0)).exp)
        v = (b * x.log.get_or_else(Real.0)).exp
        w = u * v
    }
}

/// A power of a negated exponent is the reciprocal power (value form).
theorem rpow_neg(x: Real, a: Real, u: Real) {
    x > Real.0 and x.rpow(a) = Option.some(u)
    implies x.rpow(-a) = Option.some(Real.1 / u)
} by {
    if x > Real.0 and x.rpow(a) = Option.some(u) {
        x.rpow(a) = Option.some((a * x.log.get_or_else(Real.0)).exp)
        Option.some(u) = Option.some((a * x.log.get_or_else(Real.0)).exp)
        some_injective[Real](u, (a * x.log.get_or_else(Real.0)).exp)
        u = (a * x.log.get_or_else(Real.0)).exp
        x.rpow(-a) = Option.some(((-a) * x.log.get_or_else(Real.0)).exp)
        mul_neg_left(a, x.log.get_or_else(Real.0))
        (-a) * x.log.get_or_else(Real.0) = -(a * x.log.get_or_else(Real.0))
        ((-a) * x.log.get_or_else(Real.0)).exp = (-(a * x.log.get_or_else(Real.0))).exp
        exp_neg(a * x.log.get_or_else(Real.0))
        (-(a * x.log.get_or_else(Real.0))).exp = Real.1 / (a * x.log.get_or_else(Real.0)).exp
        ((-a) * x.log.get_or_else(Real.0)).exp = Real.1 / (a * x.log.get_or_else(Real.0)).exp
        ((-a) * x.log.get_or_else(Real.0)).exp = Real.1 / u
        x.rpow(-a) = Option.some(Real.1 / u)
    }
}

/// A power of a difference of exponents is the quotient of powers (value form).
theorem rpow_sub(x: Real, a: Real, b: Real, u: Real, v: Real) {
    x > Real.0 and x.rpow(a) = Option.some(u) and x.rpow(b) = Option.some(v)
    implies x.rpow(a - b) = Option.some(u / v)
} by {
    if x > Real.0 and x.rpow(a) = Option.some(u) and x.rpow(b) = Option.some(v) {
        x.rpow(a) = Option.some((a * x.log.get_or_else(Real.0)).exp)
        Option.some(u) = Option.some((a * x.log.get_or_else(Real.0)).exp)
        some_injective[Real](u, (a * x.log.get_or_else(Real.0)).exp)
        u = (a * x.log.get_or_else(Real.0)).exp
        x.rpow(b) = Option.some((b * x.log.get_or_else(Real.0)).exp)
        Option.some(v) = Option.some((b * x.log.get_or_else(Real.0)).exp)
        some_injective[Real](v, (b * x.log.get_or_else(Real.0)).exp)
        v = (b * x.log.get_or_else(Real.0)).exp
        x.rpow(a - b) = Option.some(((a - b) * x.log.get_or_else(Real.0)).exp)
        (a - b) * x.log.get_or_else(Real.0) = a * x.log.get_or_else(Real.0) - b * x.log.get_or_else(Real.0)
        ((a - b) * x.log.get_or_else(Real.0)).exp = (a * x.log.get_or_else(Real.0) - b * x.log.get_or_else(Real.0)).exp
        exp_sub(a * x.log.get_or_else(Real.0), b * x.log.get_or_else(Real.0))
        (a * x.log.get_or_else(Real.0) - b * x.log.get_or_else(Real.0)).exp = (a * x.log.get_or_else(Real.0)).exp / (b * x.log.get_or_else(Real.0)).exp
        ((a - b) * x.log.get_or_else(Real.0)).exp = (a * x.log.get_or_else(Real.0)).exp / (b * x.log.get_or_else(Real.0)).exp
        ((a - b) * x.log.get_or_else(Real.0)).exp = u / v
        x.rpow(a - b) = Option.some(u / v)
    }
}

// =====================================================================
// Powers of special bases
// =====================================================================

/// Every real power of one is one.
theorem rpow_of_one(a: Real) {
    (Real.1).rpow(a) = Option.some(Real.1)
} by {
    Real.1 > Real.0
    (Real.1).rpow(a) = Option.some((a * (Real.1).log.get_or_else(Real.0)).exp)
    log_value_one
    (Real.1).log.get_or_else(Real.0) = Real.0
    a * (Real.1).log.get_or_else(Real.0) = a * Real.0
    a * Real.0 = Real.0
    (a * (Real.1).log.get_or_else(Real.0)).exp = (Real.0).exp
    exp_zero
    (Real.0).exp = Real.1
    (a * (Real.1).log.get_or_else(Real.0)).exp = Real.1
    (Real.1).rpow(a) = Option.some(Real.1)
}

/// A real power of Euler's number is the exponential of the exponent.
theorem rpow_e(a: Real) {
    (Real.e).rpow(a) = Option.some(a.exp)
} by {
    exp_pos(Real.1)
    (Real.1).exp > Real.0
    Real.e = (Real.1).exp
    Real.e > Real.0
    (Real.e).rpow(a) = Option.some((a * (Real.e).log.get_or_else(Real.0)).exp)
    log_value_e
    (Real.e).log.get_or_else(Real.0) = Real.1
    a * (Real.e).log.get_or_else(Real.0) = a * Real.1
    a * Real.1 = a
    (a * (Real.e).log.get_or_else(Real.0)).exp = a.exp
    (Real.e).rpow(a) = Option.some(a.exp)
}

/// A natural real power of Euler's number is the exponential of the natural.
theorem rpow_e_nat(n: Nat) {
    (Real.e).rpow(from_nat[Real](n)) = Option.some((from_nat[Real](n)).exp)
} by {
    rpow_e(from_nat[Real](n))
    (Real.e).rpow(from_nat[Real](n)) = Option.some((from_nat[Real](n)).exp)
}

// =====================================================================
// Strict bounds for real powers
// =====================================================================

/// A positive power of a base above one is strictly above one.
theorem rpow_gt_one_strict(x: Real, a: Real, u: Real) {
    x > Real.1 and a > Real.0 and x.rpow(a) = Option.some(u)
    implies u > Real.1
} by {
    if x > Real.1 and a > Real.0 and x.rpow(a) = Option.some(u) {
        Real.0 < Real.1
        lt_trans(Real.0, Real.1, x)
        Real.0 < x
        x > Real.0
        log_some_of_pos_exists(x)
        let lx: Real satisfy {
            x.log = Option.some(lx)
        }
        x.log.get_or_else(Real.0) = lx
        x.log = Option.some(x.log.get_or_else(Real.0))
        log_pos_of_gt_one(x, x.log.get_or_else(Real.0))
        x.log.get_or_else(Real.0) > Real.0
        gt_zero_imp_pos(a)
        a.is_positive
        gt_zero_imp_pos(x.log.get_or_else(Real.0))
        x.log.get_or_else(Real.0).is_positive
        mul_pos_pos(a, x.log.get_or_else(Real.0))
        a * x.log.get_or_else(Real.0) > Real.0
        exp_increasing(Real.0, a * x.log.get_or_else(Real.0))
        (Real.0).exp < (a * x.log.get_or_else(Real.0)).exp
        exp_zero
        (Real.0).exp = Real.1
        Real.1 < (a * x.log.get_or_else(Real.0)).exp
        x.rpow(a) = Option.some((a * x.log.get_or_else(Real.0)).exp)
        Option.some(u) = Option.some((a * x.log.get_or_else(Real.0)).exp)
        some_injective[Real](u, (a * x.log.get_or_else(Real.0)).exp)
        u = (a * x.log.get_or_else(Real.0)).exp
        Real.1 < u
        u > Real.1
    }
}

/// A positive power of a base between zero and one is strictly below one.
theorem rpow_lt_one_strict(x: Real, a: Real, u: Real) {
    x > Real.0 and x < Real.1 and a > Real.0 and x.rpow(a) = Option.some(u)
    implies u < Real.1
} by {
    if x > Real.0 and x < Real.1 and a > Real.0 and x.rpow(a) = Option.some(u) {
        log_some_of_pos_exists(x)
        let lx: Real satisfy {
            x.log = Option.some(lx)
        }
        x.log.get_or_else(Real.0) = lx
        x.log = Option.some(x.log.get_or_else(Real.0))
        log_neg_of_pos_lt_one(x, x.log.get_or_else(Real.0))
        x.log.get_or_else(Real.0) < Real.0
        gt_zero_imp_pos(a)
        a.is_positive
        lt_mul_pos_left(x.log.get_or_else(Real.0), Real.0, a)
        a * x.log.get_or_else(Real.0) < a * Real.0
        a * Real.0 = Real.0
        a * x.log.get_or_else(Real.0) < Real.0
        exp_increasing(a * x.log.get_or_else(Real.0), Real.0)
        (a * x.log.get_or_else(Real.0)).exp < (Real.0).exp
        exp_zero
        (Real.0).exp = Real.1
        (a * x.log.get_or_else(Real.0)).exp < Real.1
        x.rpow(a) = Option.some((a * x.log.get_or_else(Real.0)).exp)
        Option.some(u) = Option.some((a * x.log.get_or_else(Real.0)).exp)
        some_injective[Real](u, (a * x.log.get_or_else(Real.0)).exp)
        u = (a * x.log.get_or_else(Real.0)).exp
        u < Real.1
    }
}
