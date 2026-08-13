/// The characterization of intervals: a set closed under intermediate values
/// (an order-convex set) that is bounded above and below is one of the four
/// interval forms between its infimum and supremum.
///
/// This is the elementary version of the statement that the connected subsets
/// of the real line are exactly the intervals.

from order import lte_trans, lte_antisymm, lt_imp_lte, not_lte_imp_gt, not_lt_imp_gte,
    lt_of_lte_of_ne, lt_of_lte_of_ne_symm
from order_set import closed_interval_set, closed_interval_set_contains_eq,
    open_interval_set, open_interval_set_contains_eq, left_open_interval_set,
    left_open_interval_set_contains_eq, right_open_interval_set,
    right_open_interval_set_contains_eq
from order import closed_interval, open_interval, left_open_interval, right_open_interval
from data.basic.set import Set, set_ext
from real.real_field import Real
from real.supremum import is_set_infimum, is_set_supremum, is_set_upper_bound,
    is_set_lower_bound, set_bound_below_supremum_not_upper, set_not_upper_bound_witness,
    set_supremum_is_upper_bound, set_member_le_supremum
from real.topology_connected_convex import is_order_convex_real_set,
    order_convex_real_set_contains_between

numerals Real

/// True if s is one of the four bounded interval forms.
define is_bounded_interval_real_set(s: Set[Real]) -> Bool {
    exists(a: Real, b: Real) {
        s = closed_interval_set(a, b) or
        s = open_interval_set(a, b) or
        s = left_open_interval_set(a, b) or
        s = right_open_interval_set(a, b)
    }
}

/// A number that is not a lower bound of a set is exceeded by some member.
theorem set_not_lower_bound_witness(s: Set[Real], bound: Real) {
    not is_set_lower_bound(s, bound) implies exists(x: Real) {
        s.contains(x) and not bound <= x
    }
} by {
    if not is_set_lower_bound(s, bound) {
        is_set_lower_bound(s, bound) = forall(x: Real) {
            s.contains(x) implies bound <= x
        }
        let x: Real satisfy {
            s.contains(x) and not bound <= x
        }
        exists(w: Real) {
            s.contains(w) and not bound <= w
        }
    }
}

/// A number strictly between the infimum and the supremum of an order-convex
/// set belongs to the set.
theorem order_convex_between_inf_sup(
    s: Set[Real], l: Real, u: Real, x: Real
) {
    is_order_convex_real_set(s) and is_set_infimum(s, l) and is_set_supremum(s, u) and
    l < x and x < u
    implies s.contains(x)
} by {
    if is_order_convex_real_set(s) and is_set_infimum(s, l) and is_set_supremum(s, u) and
       l < x and x < u {
        set_bound_below_supremum_not_upper(s, u, x)
        not is_set_upper_bound(s, x)
        set_not_upper_bound_witness(s, x)
        let y: Real satisfy {
            s.contains(y) and not y <= x
        }
        s.contains(y)
        not_lte_imp_gt[Real](y, x)
        y > x
        x < y
        if is_set_lower_bound(s, x) {
            is_set_infimum(s, l) = (is_set_lower_bound(s, l) and forall(b: Real) {
                is_set_lower_bound(s, b) implies b <= l
            })
            forall(b: Real) {
                is_set_lower_bound(s, b) implies b <= l
            }
            is_set_lower_bound(s, x) implies x <= l
            x <= l
            l < x
            false
        }
        not is_set_lower_bound(s, x)
        set_not_lower_bound_witness(s, x)
        let w: Real satisfy {
            s.contains(w) and not x <= w
        }
        s.contains(w)
        not_lte_imp_gt[Real](x, w)
        x > w
        w < x
        lt_imp_lte(w, x)
        w <= x
        lt_imp_lte(x, y)
        x <= y
        order_convex_real_set_contains_between(s, w, y, x)
        s.contains(x)
    }
}

/// An order-convex set lies inside the closed interval between its infimum
/// and its supremum.
theorem order_convex_subset_closed_interval_inf_sup(
    s: Set[Real], l: Real, u: Real, x: Real
) {
    is_set_infimum(s, l) and is_set_supremum(s, u) and s.contains(x)
    implies closed_interval_set(l, u).contains(x)
} by {
    if is_set_infimum(s, l) and is_set_supremum(s, u) and s.contains(x) {
        is_set_infimum(s, l) = (is_set_lower_bound(s, l) and forall(b: Real) {
            is_set_lower_bound(s, b) implies b <= l
        })
        is_set_lower_bound(s, l)
        is_set_lower_bound(s, l) = forall(x0: Real) {
            s.contains(x0) implies l <= x0
        }
        s.contains(x) implies l <= x
        l <= x
        set_supremum_is_upper_bound(s, u)
        is_set_upper_bound(s, u)
        is_set_upper_bound(s, u) = forall(x0: Real) {
            s.contains(x0) implies x0 <= u
        }
        s.contains(x) implies x <= u
        x <= u
        closed_interval(l, u, x)
        closed_interval_set_contains_eq(l, u, x)
        closed_interval_set(l, u).contains(x)
    }
}

/// An order-convex set that is bounded above and below is one of the four
/// interval forms between its infimum and its supremum.
theorem order_convex_bounded_real_set_is_interval(s: Set[Real], l: Real, u: Real) {
    is_order_convex_real_set(s) and is_set_infimum(s, l) and is_set_supremum(s, u)
    implies is_bounded_interval_real_set(s)
} by {
    if is_order_convex_real_set(s) and is_set_infimum(s, l) and is_set_supremum(s, u) {
        if s.contains(l) {
            if s.contains(u) {
                forall(x: Real) {
                    if closed_interval_set(l, u).contains(x) {
                        closed_interval_set_contains_eq(l, u, x)
                        closed_interval(l, u, x)
                        l <= x
                        x <= u
                        order_convex_real_set_contains_between(s, l, u, x)
                        s.contains(x)
                    }
                    if s.contains(x) {
                        order_convex_subset_closed_interval_inf_sup(s, l, u, x)
                        closed_interval_set(l, u).contains(x)
                    }
                    closed_interval_set(l, u).contains(x) = s.contains(x)
                }
                set_ext(closed_interval_set(l, u), s)
                closed_interval_set(l, u) = s
                s = closed_interval_set(l, u)
                exists(a: Real, b: Real) {
                    s = closed_interval_set(a, b) or
                    s = open_interval_set(a, b) or
                    s = left_open_interval_set(a, b) or
                    s = right_open_interval_set(a, b)
                }
            } else {
                forall(x: Real) {
                    if right_open_interval_set(l, u).contains(x) {
                        right_open_interval_set_contains_eq(l, u, x)
                        right_open_interval(l, u, x)
                        l <= x
                        x < u
                        if x = l {
                            x = l
                            s.contains(l)
                            s.contains(x)
                        } else {
                            lt_of_lte_of_ne(l, x)
                            l < x
                            order_convex_between_inf_sup(s, l, u, x)
                            s.contains(x)
                        }
                    }
                    if s.contains(x) {
                        order_convex_subset_closed_interval_inf_sup(s, l, u, x)
                        closed_interval_set(l, u).contains(x)
                        closed_interval_set_contains_eq(l, u, x)
                        closed_interval(l, u, x)
                        l <= x
                        x <= u
                        if x = u {
                            s.contains(u)
                            false
                        }
                        x != u
                        lt_of_lte_of_ne_symm(x, u)
                        x < u
                        right_open_interval(l, u, x)
                        right_open_interval_set_contains_eq(l, u, x)
                        right_open_interval_set(l, u).contains(x)
                    }
                    right_open_interval_set(l, u).contains(x) = s.contains(x)
                }
                set_ext(right_open_interval_set(l, u), s)
                right_open_interval_set(l, u) = s
                s = right_open_interval_set(l, u)
                exists(a: Real, b: Real) {
                    s = closed_interval_set(a, b) or
                    s = open_interval_set(a, b) or
                    s = left_open_interval_set(a, b) or
                    s = right_open_interval_set(a, b)
                }
            }
        } else {
            if s.contains(u) {
                forall(x: Real) {
                    if left_open_interval_set(l, u).contains(x) {
                        left_open_interval_set_contains_eq(l, u, x)
                        left_open_interval(l, u, x)
                        l < x
                        x <= u
                        if x = u {
                            x = u
                            s.contains(u)
                            s.contains(x)
                        } else {
                            lt_of_lte_of_ne_symm(x, u)
                            x < u
                            order_convex_between_inf_sup(s, l, u, x)
                            s.contains(x)
                        }
                    }
                    if s.contains(x) {
                        order_convex_subset_closed_interval_inf_sup(s, l, u, x)
                        closed_interval_set(l, u).contains(x)
                        closed_interval_set_contains_eq(l, u, x)
                        closed_interval(l, u, x)
                        l <= x
                        x <= u
                        if x = l {
                            s.contains(l)
                            false
                        }
                        x != l
                        lt_of_lte_of_ne(x, l)
                        l < x
                        left_open_interval(l, u, x)
                        left_open_interval_set_contains_eq(l, u, x)
                        left_open_interval_set(l, u).contains(x)
                    }
                    left_open_interval_set(l, u).contains(x) = s.contains(x)
                }
                set_ext(left_open_interval_set(l, u), s)
                left_open_interval_set(l, u) = s
                s = left_open_interval_set(l, u)
                exists(a: Real, b: Real) {
                    s = closed_interval_set(a, b) or
                    s = open_interval_set(a, b) or
                    s = left_open_interval_set(a, b) or
                    s = right_open_interval_set(a, b)
                }
            } else {
                forall(x: Real) {
                    if open_interval_set(l, u).contains(x) {
                        open_interval_set_contains_eq(l, u, x)
                        open_interval(l, u, x)
                        l < x
                        x < u
                        order_convex_between_inf_sup(s, l, u, x)
                        s.contains(x)
                    }
                    if s.contains(x) {
                        order_convex_subset_closed_interval_inf_sup(s, l, u, x)
                        closed_interval_set(l, u).contains(x)
                        closed_interval_set_contains_eq(l, u, x)
                        closed_interval(l, u, x)
                        l <= x
                        x <= u
                        if x = l {
                            s.contains(l)
                            false
                        }
                        x != l
                        lt_of_lte_of_ne(x, l)
                        l < x
                        if x = u {
                            s.contains(u)
                            false
                        }
                        x != u
                        lt_of_lte_of_ne_symm(x, u)
                        x < u
                        open_interval(l, u, x)
                        open_interval_set_contains_eq(l, u, x)
                        open_interval_set(l, u).contains(x)
                    }
                    open_interval_set(l, u).contains(x) = s.contains(x)
                }
                set_ext(open_interval_set(l, u), s)
                open_interval_set(l, u) = s
                s = open_interval_set(l, u)
                exists(a: Real, b: Real) {
                    s = closed_interval_set(a, b) or
                    s = open_interval_set(a, b) or
                    s = left_open_interval_set(a, b) or
                    s = right_open_interval_set(a, b)
                }
            }
        }
    }
}
