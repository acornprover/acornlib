/// Named reciprocal and quotient examples using the global real calculus quotient API.

from data.basic.function_algebra import pointwise_add, pointwise_neg, pointwise_mul
from data.basic.functions import identity_fn
from real.real_base import Real
from real.continuity_affine import affine_real
from real.continuity_square import square_real
from real.calculus_api import is_derivative_fn, differentiable_everywhere
from real.calculus_api_standard_examples import derivative_fn_affine_real,
    differentiable_everywhere_affine_real, derivative_fn_square_real,
    differentiable_everywhere_square_real
from real.calculus_quotient_api import nonvanishing_everywhere,
    derivative_fn_reciprocal, derivative_fn_quotient,
    differentiable_everywhere_reciprocal, differentiable_everywhere_quotient
from real.derivative_quotient import pointwise_div_real, pointwise_reciprocal_real

/// The reciprocal of a named affine function has the quotient-API derivative shape.
theorem derivative_fn_reciprocal_affine_real(a: Real, b: Real) {
    nonvanishing_everywhere(affine_real(a, b)) implies is_derivative_fn(
        pointwise_reciprocal_real(affine_real(a, b)),
        pointwise_mul(
            pointwise_div_real(
                constant[Real, Real](-Real.1),
                pointwise_mul(affine_real(a, b), affine_real(a, b))
            ),
            constant[Real, Real](a)
        )
    )
} by {
    if nonvanishing_everywhere(affine_real(a, b)) {
        derivative_fn_affine_real(a, b)
        derivative_fn_reciprocal(affine_real(a, b), constant[Real, Real](a))
        is_derivative_fn(
            pointwise_reciprocal_real(affine_real(a, b)),
            pointwise_mul(
                pointwise_div_real(
                    constant[Real, Real](-Real.1),
                    pointwise_mul(affine_real(a, b), affine_real(a, b))
                ),
                constant[Real, Real](a)
            )
        )
    }
}

/// The reciprocal of a nonvanishing named affine function is differentiable everywhere.
theorem differentiable_everywhere_reciprocal_affine_real(a: Real, b: Real) {
    nonvanishing_everywhere(affine_real(a, b)) implies
        differentiable_everywhere(pointwise_reciprocal_real(affine_real(a, b)))
} by {
    if nonvanishing_everywhere(affine_real(a, b)) {
        differentiable_everywhere_affine_real(a, b)
        differentiable_everywhere_reciprocal(affine_real(a, b))
        differentiable_everywhere(pointwise_reciprocal_real(affine_real(a, b)))
    }
}

/// The reciprocal of the named square function has the quotient-API derivative shape.
theorem derivative_fn_reciprocal_square_real {
    nonvanishing_everywhere(square_real) implies is_derivative_fn(
        pointwise_reciprocal_real(square_real),
        pointwise_mul(
            pointwise_div_real(
                constant[Real, Real](-Real.1),
                pointwise_mul(square_real, square_real)
            ),
            pointwise_add(identity_fn[Real], identity_fn[Real])
        )
    )
} by {
    if nonvanishing_everywhere(square_real) {
        derivative_fn_square_real
        derivative_fn_reciprocal(square_real, pointwise_add(identity_fn[Real], identity_fn[Real]))
        is_derivative_fn(
            pointwise_reciprocal_real(square_real),
            pointwise_mul(
                pointwise_div_real(
                    constant[Real, Real](-Real.1),
                    pointwise_mul(square_real, square_real)
                ),
                pointwise_add(identity_fn[Real], identity_fn[Real])
            )
        )
    }
}

/// The reciprocal of a nonvanishing named square function is differentiable everywhere.
theorem differentiable_everywhere_reciprocal_square_real {
    nonvanishing_everywhere(square_real) implies
        differentiable_everywhere(pointwise_reciprocal_real(square_real))
} by {
    if nonvanishing_everywhere(square_real) {
        differentiable_everywhere_square_real
        differentiable_everywhere_reciprocal(square_real)
        differentiable_everywhere(pointwise_reciprocal_real(square_real))
    }
}

/// A quotient of two named affine functions has the global quotient-rule derivative shape.
theorem derivative_fn_affine_over_affine(a: Real, b: Real, c: Real, d: Real) {
    nonvanishing_everywhere(affine_real(c, d)) implies is_derivative_fn(
        pointwise_div_real(affine_real(a, b), affine_real(c, d)),
        pointwise_div_real(
            pointwise_add(
                pointwise_mul(constant[Real, Real](a), affine_real(c, d)),
                pointwise_neg(pointwise_mul(affine_real(a, b), constant[Real, Real](c)))
            ),
            pointwise_mul(affine_real(c, d), affine_real(c, d))
        )
    )
} by {
    if nonvanishing_everywhere(affine_real(c, d)) {
        derivative_fn_affine_real(a, b)
        derivative_fn_affine_real(c, d)
        derivative_fn_quotient(
            affine_real(a, b),
            affine_real(c, d),
            constant[Real, Real](a),
            constant[Real, Real](c)
        )
        is_derivative_fn(
            pointwise_div_real(affine_real(a, b), affine_real(c, d)),
            pointwise_div_real(
                pointwise_add(
                    pointwise_mul(constant[Real, Real](a), affine_real(c, d)),
                    pointwise_neg(pointwise_mul(affine_real(a, b), constant[Real, Real](c)))
                ),
                pointwise_mul(affine_real(c, d), affine_real(c, d))
            )
        )
    }
}

/// A quotient of two named affine functions is differentiable everywhere under a nonvanishing denominator.
theorem differentiable_everywhere_affine_over_affine(a: Real, b: Real, c: Real, d: Real) {
    nonvanishing_everywhere(affine_real(c, d)) implies
        differentiable_everywhere(pointwise_div_real(affine_real(a, b), affine_real(c, d)))
} by {
    if nonvanishing_everywhere(affine_real(c, d)) {
        differentiable_everywhere_affine_real(a, b)
        differentiable_everywhere_affine_real(c, d)
        differentiable_everywhere_quotient(affine_real(a, b), affine_real(c, d))
        differentiable_everywhere(pointwise_div_real(affine_real(a, b), affine_real(c, d)))
    }
}

/// The quotient of the square function by a named affine denominator has the global quotient-rule derivative shape.
theorem derivative_fn_square_over_affine(a: Real, b: Real) {
    nonvanishing_everywhere(affine_real(a, b)) implies is_derivative_fn(
        pointwise_div_real(square_real, affine_real(a, b)),
        pointwise_div_real(
            pointwise_add(
                pointwise_mul(pointwise_add(identity_fn[Real], identity_fn[Real]), affine_real(a, b)),
                pointwise_neg(pointwise_mul(square_real, constant[Real, Real](a)))
            ),
            pointwise_mul(affine_real(a, b), affine_real(a, b))
        )
    )
} by {
    if nonvanishing_everywhere(affine_real(a, b)) {
        derivative_fn_square_real
        derivative_fn_affine_real(a, b)
        derivative_fn_quotient(
            square_real,
            affine_real(a, b),
            pointwise_add(identity_fn[Real], identity_fn[Real]),
            constant[Real, Real](a)
        )
        is_derivative_fn(
            pointwise_div_real(square_real, affine_real(a, b)),
            pointwise_div_real(
                pointwise_add(
                    pointwise_mul(pointwise_add(identity_fn[Real], identity_fn[Real]), affine_real(a, b)),
                    pointwise_neg(pointwise_mul(square_real, constant[Real, Real](a)))
                ),
                pointwise_mul(affine_real(a, b), affine_real(a, b))
            )
        )
    }
}

/// The quotient of the square function by a nonvanishing named affine denominator is differentiable everywhere.
theorem differentiable_everywhere_square_over_affine(a: Real, b: Real) {
    nonvanishing_everywhere(affine_real(a, b)) implies
        differentiable_everywhere(pointwise_div_real(square_real, affine_real(a, b)))
} by {
    if nonvanishing_everywhere(affine_real(a, b)) {
        differentiable_everywhere_square_real
        differentiable_everywhere_affine_real(a, b)
        differentiable_everywhere_quotient(square_real, affine_real(a, b))
        differentiable_everywhere(pointwise_div_real(square_real, affine_real(a, b)))
    }
}

/// The quotient of a named affine numerator by the square function has the global quotient-rule derivative shape.
theorem derivative_fn_affine_over_square(a: Real, b: Real) {
    nonvanishing_everywhere(square_real) implies is_derivative_fn(
        pointwise_div_real(affine_real(a, b), square_real),
        pointwise_div_real(
            pointwise_add(
                pointwise_mul(constant[Real, Real](a), square_real),
                pointwise_neg(pointwise_mul(affine_real(a, b), pointwise_add(identity_fn[Real], identity_fn[Real])))
            ),
            pointwise_mul(square_real, square_real)
        )
    )
} by {
    if nonvanishing_everywhere(square_real) {
        derivative_fn_affine_real(a, b)
        derivative_fn_square_real
        derivative_fn_quotient(
            affine_real(a, b),
            square_real,
            constant[Real, Real](a),
            pointwise_add(identity_fn[Real], identity_fn[Real])
        )
        is_derivative_fn(
            pointwise_div_real(affine_real(a, b), square_real),
            pointwise_div_real(
                pointwise_add(
                    pointwise_mul(constant[Real, Real](a), square_real),
                    pointwise_neg(pointwise_mul(affine_real(a, b), pointwise_add(identity_fn[Real], identity_fn[Real])))
                ),
                pointwise_mul(square_real, square_real)
            )
        )
    }
}

/// The quotient of a named affine numerator by a nonvanishing square denominator is differentiable everywhere.
theorem differentiable_everywhere_affine_over_square(a: Real, b: Real) {
    nonvanishing_everywhere(square_real) implies
        differentiable_everywhere(pointwise_div_real(affine_real(a, b), square_real))
} by {
    if nonvanishing_everywhere(square_real) {
        differentiable_everywhere_affine_real(a, b)
        differentiable_everywhere_square_real
        differentiable_everywhere_quotient(affine_real(a, b), square_real)
        differentiable_everywhere(pointwise_div_real(affine_real(a, b), square_real))
    }
}
