/// Term-vanishing consumers for geometric-majorant real series.
///
/// This module is a small downstream layer on top of the published
/// `real.series_geometric_majorant` endpoints. It turns the resulting series
/// convergence facts into term convergence-to-zero and vanishing facts without
/// adding any new eventual predicate or public growth-rate notation.

from list import partial
from nat import Nat
from real.abs_conv import absolutely_converges, absolutely_converges_imp_converges, abs_fn
from real.asymptotic_bounds import converges_to_zero_imp_vanishes
from real.limits import vanishes
from real.real_seq import converges_to
from real.real_series import converges, series_conv_imp_term_vanishes, tail
from real.series_geometric_majorant import geometric_majorant, geometric_majorant_series_converges,
    eventually_abs_le_geometric_series_converges, eventually_nonneg_le_geometric_series_converges,
    tail_abs_le_geometric_absolutely_converges, tail_nonneg_le_geometric_series_converges
from real.real_base import Real

numerals Real

/// The terms of a convergent geometric majorant converge to zero.
theorem geometric_majorant_terms_converge_to_zero(c: Real, r: Real) {
    Real.0 <= r and r < Real.1 implies converges_to(geometric_majorant(c, r), Real.0)
} by {
    if Real.0 <= r and r < Real.1 {
        geometric_majorant_series_converges(c, r)
        converges(partial(geometric_majorant(c, r)))
        series_conv_imp_term_vanishes(geometric_majorant(c, r))
        converges_to(geometric_majorant(c, r), Real.0)
    }
}

/// The terms of a convergent geometric majorant vanish.
theorem geometric_majorant_terms_vanish(c: Real, r: Real) {
    Real.0 <= r and r < Real.1 implies vanishes(geometric_majorant(c, r))
} by {
    if Real.0 <= r and r < Real.1 {
        geometric_majorant_terms_converge_to_zero(c, r)
        converges_to(geometric_majorant(c, r), Real.0)
        converges_to_zero_imp_vanishes(geometric_majorant(c, r))
        vanishes(geometric_majorant(c, r))
    }
}

/// Eventual absolute domination by a geometric majorant forces terms to tend to zero.
theorem eventually_abs_le_geometric_terms_converge_to_zero(f: Nat -> Real, c: Real, r: Real, n: Nat) {
    Real.0 <= c
    and Real.0 <= r
    and r < Real.1
    and (forall(k: Nat) { n <= k implies abs_fn(f)(k) <= geometric_majorant(c, r, k) })
    implies converges_to(f, Real.0)
} by {
    if Real.0 <= c
        and Real.0 <= r
        and r < Real.1
        and (forall(k: Nat) { n <= k implies abs_fn(f)(k) <= geometric_majorant(c, r, k) }) {
        eventually_abs_le_geometric_series_converges(f, c, r, n)
        converges(partial(f))
        series_conv_imp_term_vanishes(f)
        converges_to(f, Real.0)
    }
}

/// Eventual absolute domination by a geometric majorant forces vanishing terms.
theorem eventually_abs_le_geometric_terms_vanish(f: Nat -> Real, c: Real, r: Real, n: Nat) {
    Real.0 <= c
    and Real.0 <= r
    and r < Real.1
    and (forall(k: Nat) { n <= k implies abs_fn(f)(k) <= geometric_majorant(c, r, k) })
    implies vanishes(f)
} by {
    if Real.0 <= c
        and Real.0 <= r
        and r < Real.1
        and (forall(k: Nat) { n <= k implies abs_fn(f)(k) <= geometric_majorant(c, r, k) }) {
        eventually_abs_le_geometric_terms_converge_to_zero(f, c, r, n)
        converges_to(f, Real.0)
        converges_to_zero_imp_vanishes(f)
        vanishes(f)
    }
}

/// Eventual nonnegative domination by a geometric majorant forces terms to tend to zero.
theorem eventually_nonneg_le_geometric_terms_converge_to_zero(f: Nat -> Real, c: Real, r: Real, n: Nat) {
    Real.0 <= c
    and Real.0 <= r
    and r < Real.1
    and (forall(k: Nat) { n <= k implies Real.0 <= f(k) })
    and (forall(k: Nat) { n <= k implies f(k) <= geometric_majorant(c, r, k) })
    implies converges_to(f, Real.0)
} by {
    if Real.0 <= c
        and Real.0 <= r
        and r < Real.1
        and (forall(k: Nat) { n <= k implies Real.0 <= f(k) })
        and (forall(k: Nat) { n <= k implies f(k) <= geometric_majorant(c, r, k) }) {
        eventually_nonneg_le_geometric_series_converges(f, c, r, n)
        converges(partial(f))
        series_conv_imp_term_vanishes(f)
        converges_to(f, Real.0)
    }
}

/// Eventual nonnegative domination by a geometric majorant forces vanishing terms.
theorem eventually_nonneg_le_geometric_terms_vanish(f: Nat -> Real, c: Real, r: Real, n: Nat) {
    Real.0 <= c
    and Real.0 <= r
    and r < Real.1
    and (forall(k: Nat) { n <= k implies Real.0 <= f(k) })
    and (forall(k: Nat) { n <= k implies f(k) <= geometric_majorant(c, r, k) })
    implies vanishes(f)
} by {
    if Real.0 <= c
        and Real.0 <= r
        and r < Real.1
        and (forall(k: Nat) { n <= k implies Real.0 <= f(k) })
        and (forall(k: Nat) { n <= k implies f(k) <= geometric_majorant(c, r, k) }) {
        eventually_nonneg_le_geometric_terms_converge_to_zero(f, c, r, n)
        converges_to(f, Real.0)
        converges_to_zero_imp_vanishes(f)
        vanishes(f)
    }
}

/// Tail-form absolute domination by a geometric majorant forces vanishing terms.
theorem tail_abs_le_geometric_terms_vanish(f: Nat -> Real, c: Real, r: Real, n: Nat) {
    Real.0 <= c
    and Real.0 <= r
    and r < Real.1
    and (forall(k: Nat) { abs_fn(tail(f, n))(k) <= geometric_majorant(c, r, k) })
    implies vanishes(f)
} by {
    if Real.0 <= c
        and Real.0 <= r
        and r < Real.1
        and (forall(k: Nat) { abs_fn(tail(f, n))(k) <= geometric_majorant(c, r, k) }) {
        tail_abs_le_geometric_absolutely_converges(f, c, r, n)
        absolutely_converges(f)
        absolutely_converges_imp_converges(f)
        converges(partial(f))
        series_conv_imp_term_vanishes(f)
        converges_to(f, Real.0)
        converges_to_zero_imp_vanishes(f)
        vanishes(f)
    }
}

/// Tail-form nonnegative domination by a geometric majorant forces vanishing terms.
theorem tail_nonneg_le_geometric_terms_vanish(f: Nat -> Real, c: Real, r: Real, n: Nat) {
    Real.0 <= c
    and Real.0 <= r
    and r < Real.1
    and (forall(k: Nat) { Real.0 <= tail(f, n)(k) })
    and (forall(k: Nat) { tail(f, n)(k) <= geometric_majorant(c, r, k) })
    implies vanishes(f)
} by {
    if Real.0 <= c
        and Real.0 <= r
        and r < Real.1
        and (forall(k: Nat) { Real.0 <= tail(f, n)(k) })
        and (forall(k: Nat) { tail(f, n)(k) <= geometric_majorant(c, r, k) }) {
        tail_nonneg_le_geometric_series_converges(f, c, r, n)
        converges(partial(f))
        series_conv_imp_term_vanishes(f)
        converges_to(f, Real.0)
        converges_to_zero_imp_vanishes(f)
        vanishes(f)
    }
}
