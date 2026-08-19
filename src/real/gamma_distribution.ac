/// The Gamma distribution on natural (Erlang) shape parameters.
///
/// The Gamma density with shape k ∈ ℕ and unit rate is
///
///     f_k(t) = t^k·e^(-t) / k!   for t ≥ 0,
///
/// whose normalization is the factorial Γ(k + 1) = k! (in the classical
/// notation Γ(α) = ∫₀^∞ t^(α-1)·e^(-t) dt, the shape α = k + 1 has
/// Γ(α) = (α - 1)! = k!).
///
/// The library has only finite-interval Riemann integrals, so this file
/// proves the finite-interval versions and records the improper limits:
///
///   - gamma_interval(0, 0, b) = ∫₀^b e^(-t) dt = 1 - e^(-b) → 1,
///   - gamma_interval(1, 0, b) = ∫₀^b t·e^(-t) dt = 1 - (b + 1)·e^(-b) → 1,
///   - gamma_interval(2, 0, b) = ∫₀^b t²·e^(-t) dt = 2 - (b² + 2b + 2)·e^(-b) → 2
///     (the k = 2 development is commented out below as future work),
///
/// so the normalized densities integrate to one in the limit.  The k = 0 and
/// k = 1 cases reuse the exponential machinery of real.exponential_distribution
/// (rate one).

from nat import Nat, from_nat, from_nat_one, pow_one, lte_one_factorial
from order import lt_imp_lte, lte_trans, lt_imp_ne, lte_lt_trans
from data.basic.functions import identity_fn, function_extensionality, function_eq_transport_predicate_rev
from data.basic.function_algebra import pointwise_mul, pointwise_neg, pointwise_add
from data.basic.logic import eq_true_intro
from real.real_field import Real
from real.gamma import neg_fn, exp_neg_fn, gamma_fn, gamma_interval, exp_neg_fn_derivative, exp_neg_anti_derivative
from real.exp import exp_zero, two, two_positive, exp_pos
from real.integral_exp import exp_lte_mono, exp_continuous, from_nat_lte_mono
from real.continuity_base import continuous
from real.continuity_sequences import identity_function_is_continuous
from real.continuity_const_mul import const_mul_left, continuous_const_mul_left
from real.continuity_composition import continuous_compose, constant_function_is_continuous
from real.continuity_pointwise import continuous_pointwise_neg, continuous_pointwise_add
from real.continuity_pointwise_mul import continuous_pointwise_mul
from real.calculus_api import is_derivative_fn, derivative_fn_identity, derivative_fn_constant, derivative_fn_const_mul, derivative_fn_neg, derivative_fn_add, derivative_fn_mul, derivative_fn_square
from real.integral import integral, is_integrable, interval_contains, interval_contains_left, interval_contains_right
from real.integral_trig import fn_integrable_gen
from real.integral_exp import ftc2_general
from real.distribution_common import fn_integrable_gen_derivative_bound, integral_ftc2_derivative_bound, mul_nonneg_lte, mul_le_mul_nonneg_lte, is_derivative_fn_pointwise_add_comm
from real.integral_polynomial_values import integral_and_integrable_eq_on
from real.distribution_common import is_derivative_fn_both_eq
from real.exponential_distribution import exp_decay, exp_rate_density, exp_rate_density_deriv, exp_rate_anti, exp_rate_density_integrable, exp_rate_mass_interval, exponential_moment_integrand, exponential_moment_deriv, exponential_moment_anti, exponential_moment_integrable, exponential_mean_interval, exponential_moment_lower_bound_on, exponential_moment_upper_bound_on, exp_neg_fn_continuous
from real.real_ring import real_mul_comm, mul_assoc, mul_abs, mul_nonneg, mul_le_mul_nonneg, square_nonneg
from real.real_base import abs_gte_zero, abs_neg, gt_zero_imp_pos, pos_gt_zero
from real.derivative_trig import abs_of_nonneg
from ordered_field import mul_le_mul_of_nonneg_right, zero_is_smaller_than_one, inverse_of_positive_is_positive
from algebra.add_ordered_group import add_le_add_right, add_le_add, neg_le_neg
from real.real_series import pow_nonneg, triangle_ineq
from real.derivative_continuity import div_mul_cancel_denominator
from real.continuity_pow import pow_real_fn, continuous_pow_real_fn

numerals Real
numerals Nat

// ---------------------------------------------------------------------------
// The Erlang density
// ---------------------------------------------------------------------------

/// The Erlang density with natural shape k and unit rate:
/// t ↦ t^k·e^(-t)/k! on the nonnegative half-line.
define erlang_density(k: Nat, t: Real) -> Real {
    gamma_fn(k, t) / from_nat[Real](k.factorial)
}

/// The factorial of zero is one.
lemma factorial_real_zero {
    from_nat[Real](Nat.0.factorial) = Real.1
} by {
    Nat.0.factorial = Nat.1
    from_nat[Real](Nat.0.factorial) = from_nat[Real](Nat.1)
    from_nat_one[Real]
    from_nat[Real](Nat.1) = Real.1
    from_nat[Real](Nat.0.factorial) = Real.1
}

/// The factorial of one is one.
lemma factorial_real_one {
    from_nat[Real](Nat.1.factorial) = Real.1
} by {
    Nat.1.factorial = Nat.1
    from_nat[Real](Nat.1.factorial) = from_nat[Real](Nat.1)
    from_nat_one[Real]
    from_nat[Real](Nat.1) = Real.1
    from_nat[Real](Nat.1.factorial) = Real.1
}

/// e^(-t) is strictly positive.
theorem exp_neg_fn_pos(t: Real) {
    exp_neg_fn(t) > Real.0
} by {
    exp_pos(-t)
    (-t).exp > Real.0
    exp_neg_fn(t) = (neg_fn(t)).exp
    neg_fn(t) = -t
    exp_neg_fn(t) = (-t).exp
    exp_neg_fn(t) > Real.0
}

/// e^(-t) is nonnegative.
theorem exp_neg_fn_nonneg(t: Real) {
    Real.0 <= exp_neg_fn(t)
} by {
    exp_neg_fn_pos(t)
    exp_neg_fn(t) > Real.0
    lt_imp_lte(Real.0, exp_neg_fn(t))
    Real.0 <= exp_neg_fn(t)
}

/// e^(-t) is at most one for t ≥ 0.
theorem exp_neg_fn_le_one(t: Real) {
    Real.0 <= t implies exp_neg_fn(t) <= Real.1
} by {
    if Real.0 <= t {
        neg_le_neg(Real.0, t)
        -t <= -Real.0
        -Real.0 = Real.0
        -t <= Real.0
        exp_lte_mono(-t, Real.0)
        (-t).exp <= (Real.0).exp
        exp_zero
        (Real.0).exp = Real.1
        (-t).exp <= Real.1
        exp_neg_fn(t) = (neg_fn(t)).exp
        neg_fn(t) = -t
        exp_neg_fn(t) = (-t).exp
        exp_neg_fn(t) <= Real.1
    }
}

/// The Gamma integrand t^n·e^(-t) is nonnegative for t ≥ 0.
theorem gamma_fn_nonneg(n: Nat, t: Real) {
    Real.0 <= t implies Real.0 <= gamma_fn(n, t)
} by {
    if Real.0 <= t {
        gamma_fn(n, t) = pow_real_fn(n, t) * exp_neg_fn(t)
        pow_nonneg(t, n)
        Real.0 <= t.pow(n)
        pow_real_fn(n, t) = t.pow(n)
        Real.0 <= pow_real_fn(n, t)
        exp_neg_fn_nonneg(t)
        Real.0 <= exp_neg_fn(t)
        mul_nonneg_lte(pow_real_fn(n, t), exp_neg_fn(t))
        Real.0 <= pow_real_fn(n, t) * exp_neg_fn(t)
        Real.0 <= gamma_fn(n, t)
    }
}

/// The Erlang density is nonnegative for t ≥ 0.
theorem erlang_density_nonneg(k: Nat, t: Real) {
    Real.0 <= t implies Real.0 <= erlang_density(k, t)
} by {
    if Real.0 <= t {
        erlang_density(k, t) = gamma_fn(k, t) / from_nat[Real](k.factorial)
        gamma_fn_nonneg(k, t)
        Real.0 <= gamma_fn(k, t)
        lte_one_factorial(k.factorial)
        Nat.1 <= k.factorial
        from_nat_lte_mono(Nat.1, k.factorial)
        from_nat[Real](Nat.1) <= from_nat[Real](k.factorial)
        from_nat_one[Real]
        from_nat[Real](Nat.1) = Real.1
        Real.1 <= from_nat[Real](k.factorial)
        zero_is_smaller_than_one[Real]
        Real.0 < Real.1
        lt_imp_lte(Real.0, Real.1)
        Real.0 <= Real.1
        lte_lt_trans(Real.0, Real.1, from_nat[Real](k.factorial))
        Real.0 < from_nat[Real](k.factorial)
        gt_zero_imp_pos(from_nat[Real](k.factorial))
        from_nat[Real](k.factorial).is_positive
        inverse_of_positive_is_positive(from_nat[Real](k.factorial))
        Real.0 < from_nat[Real](k.factorial).inverse
        gt_zero_imp_pos(from_nat[Real](k.factorial).inverse)
        from_nat[Real](k.factorial).inverse.is_positive
        Real.0 <= from_nat[Real](k.factorial).inverse
        mul_nonneg_lte(gamma_fn(k, t), from_nat[Real](k.factorial).inverse)
        Real.0 <= gamma_fn(k, t) * from_nat[Real](k.factorial).inverse
        gamma_fn(k, t) / from_nat[Real](k.factorial) =
            gamma_fn(k, t) * from_nat[Real](k.factorial).inverse
        Real.0 <= gamma_fn(k, t) / from_nat[Real](k.factorial)
        Real.0 <= erlang_density(k, t)
    }
}

// ---------------------------------------------------------------------------
// The k = 0 case: the Erlang-1 density is the exponential density (rate 1)
// ---------------------------------------------------------------------------

/// The k = 0 Gamma integrand is e^(-t), the exponential decay of rate one.
theorem gamma_fn_zero_eq_exp_neg {
    gamma_fn(Nat.0) = exp_neg_fn
} by {
    forall(t: Real) {
        gamma_fn(Nat.0, t) = pow_real_fn(Nat.0, t) * exp_neg_fn(t)
        pow_real_fn(Nat.0, t) = t.pow(Nat.0)
        t.pow(Nat.0) = Real.1
        pow_real_fn(Nat.0, t) = Real.1
        Real.1 * exp_neg_fn(t) = exp_neg_fn(t)
        gamma_fn(Nat.0, t) = exp_neg_fn(t)
    }
    function_extensionality(gamma_fn(Nat.0), exp_neg_fn)
    gamma_fn(Nat.0) = exp_neg_fn
}

/// The exponential decay of rate one is e^(-t).
theorem exp_decay_one_eq_exp_neg {
    exp_decay(Real.1) = exp_neg_fn
} by {
    forall(t: Real) {
        exp_decay(Real.1, t) = (-(Real.1 * t)).exp
        Real.1 * t = t
        -(Real.1 * t) = -t
        exp_decay(Real.1, t) = (-t).exp
        exp_neg_fn(t) = (neg_fn(t)).exp
        neg_fn(t) = -t
        exp_neg_fn(t) = (-t).exp
        exp_decay(Real.1, t) = exp_neg_fn(t)
    }
    function_extensionality(exp_decay(Real.1), exp_neg_fn)
    exp_decay(Real.1) = exp_neg_fn
}

/// The k = 0 Gamma integrand is the exponential density with rate one.
theorem gamma_fn_zero_eq_exp_density {
    gamma_fn(Nat.0) = exp_rate_density(Real.1)
} by {
    forall(t: Real) {
        gamma_fn(Nat.0, t) = pow_real_fn(Nat.0, t) * exp_neg_fn(t)
        pow_real_fn(Nat.0, t) = t.pow(Nat.0)
        t.pow(Nat.0) = Real.1
        pow_real_fn(Nat.0, t) = Real.1
        Real.1 * exp_neg_fn(t) = exp_neg_fn(t)
        gamma_fn(Nat.0, t) = exp_neg_fn(t)
        exp_rate_density(Real.1, t) = Real.1 * exp_decay(Real.1, t)
        exp_decay(Real.1, t) = (-(Real.1 * t)).exp
        Real.1 * t = t
        -(Real.1 * t) = -t
        exp_decay(Real.1, t) = (-t).exp
        exp_neg_fn(t) = (neg_fn(t)).exp
        neg_fn(t) = -t
        exp_neg_fn(t) = (-t).exp
        exp_decay(Real.1, t) = exp_neg_fn(t)
        Real.1 * exp_neg_fn(t) = exp_neg_fn(t)
        exp_rate_density(Real.1, t) = exp_neg_fn(t)
        gamma_fn(Nat.0, t) = exp_rate_density(Real.1, t)
    }
    function_extensionality(gamma_fn(Nat.0), exp_rate_density(Real.1))
    gamma_fn(Nat.0) = exp_rate_density(Real.1)
}

/// The k = 0 Gamma integrand is bounded between zero and one on [0, b].
theorem gamma_fn_zero_bounds_on(b: Real, t: Real) {
    Real.0 <= b and interval_contains(Real.0, b, t)
    implies (Real.0 <= gamma_fn(Nat.0, t) and gamma_fn(Nat.0, t) <= Real.1)
} by {
    if Real.0 <= b and interval_contains(Real.0, b, t) {
        gamma_fn(Nat.0, t) = pow_real_fn(Nat.0, t) * exp_neg_fn(t)
        pow_real_fn(Nat.0, t) = t.pow(Nat.0)
        t.pow(Nat.0) = Real.1
        pow_real_fn(Nat.0, t) = Real.1
        Real.1 * exp_neg_fn(t) = exp_neg_fn(t)
        gamma_fn(Nat.0, t) = exp_neg_fn(t)
        exp_neg_fn_nonneg(t)
        Real.0 <= exp_neg_fn(t)
        Real.0 <= gamma_fn(Nat.0, t)
        interval_contains_left(Real.0, b, t)
        Real.0 <= t
        exp_neg_fn_le_one(t)
        exp_neg_fn(t) <= Real.1
        gamma_fn(Nat.0, t) <= Real.1
        Real.0 <= gamma_fn(Nat.0, t) and gamma_fn(Nat.0, t) <= Real.1
    }
}

/// The k = 0 Gamma integrand is integrable on [0, b] and its integral is
/// 1 - e^(-b).
theorem gamma_interval_zero(b: Real) {
    Real.0 <= b implies
    (is_integrable(gamma_fn(Nat.0), Real.0, b) and
     gamma_interval(Nat.0, Real.0, b) = Real.1 - exp_neg_fn(b))
} by {
    if Real.0 <= b {
        zero_is_smaller_than_one[Real]
        Real.0 < Real.1
        exp_rate_density_integrable(Real.1, Real.0, b)
        is_integrable(exp_rate_density(Real.1), Real.0, b)
        gamma_fn_zero_eq_exp_density
        gamma_fn(Nat.0) = exp_rate_density(Real.1)
        forall(t: Real) {
            if interval_contains(Real.0, b, t) {
                gamma_fn_zero_eq_exp_density
                gamma_fn(Nat.0) = exp_rate_density(Real.1)
                gamma_fn(Nat.0, t) = exp_rate_density(Real.1, t)
            }
        }
        forall(t: Real) {
            if interval_contains(Real.0, b, t) {
                gamma_fn_zero_bounds_on(b, t)
                Real.0 <= gamma_fn(Nat.0, t) and gamma_fn(Nat.0, t) <= Real.1
                Real.0 <= gamma_fn(Nat.0, t)
            }
        }
        forall(t: Real) {
            if interval_contains(Real.0, b, t) {
                gamma_fn_zero_bounds_on(b, t)
                Real.0 <= gamma_fn(Nat.0, t) and gamma_fn(Nat.0, t) <= Real.1
                gamma_fn(Nat.0, t) <= Real.1
            }
        }
        integral_and_integrable_eq_on(gamma_fn(Nat.0), exp_rate_density(Real.1),
            Real.0, b, Real.0, Real.1)
        is_integrable(gamma_fn(Nat.0), Real.0, b) and
            integral(gamma_fn(Nat.0), Real.0, b) =
                integral(exp_rate_density(Real.1), Real.0, b)
        is_integrable(gamma_fn(Nat.0), Real.0, b)
        integral(gamma_fn(Nat.0), Real.0, b) =
            integral(exp_rate_density(Real.1), Real.0, b)
        gamma_interval(Nat.0, Real.0, b) = integral(gamma_fn(Nat.0), Real.0, b)
        gamma_interval(Nat.0, Real.0, b) = integral(exp_rate_density(Real.1), Real.0, b)
        exp_rate_mass_interval(Real.1, b)
        integral(exp_rate_density(Real.1), Real.0, b) = Real.1 - exp_decay(Real.1, b)
        exp_decay_one_eq_exp_neg
        exp_decay(Real.1) = exp_neg_fn
        exp_decay(Real.1, b) = exp_neg_fn(b)
        Real.1 - exp_decay(Real.1, b) = Real.1 - exp_neg_fn(b)
        gamma_interval(Nat.0, Real.0, b) = Real.1 - exp_neg_fn(b)
        is_integrable(gamma_fn(Nat.0), Real.0, b) and
            gamma_interval(Nat.0, Real.0, b) = Real.1 - exp_neg_fn(b)
    }
}

// ---------------------------------------------------------------------------
// The k = 1 case: ∫₀^b t·e^(-t) dt = 1 - (b + 1)·e^(-b)
// ---------------------------------------------------------------------------

/// The k = 1 Gamma integrand is t·e^(-t), the exponential moment integrand
/// of rate one.
theorem gamma_fn_one_eq_moment {
    gamma_fn(Nat.1) = exponential_moment_integrand(Real.1)
} by {
    forall(t: Real) {
        gamma_fn(Nat.1, t) = pow_real_fn(Nat.1, t) * exp_neg_fn(t)
        pow_real_fn(Nat.1, t) = t.pow(Nat.1)
        t.pow(Nat.1) = t
        pow_real_fn(Nat.1, t) = t
        t * exp_neg_fn(t) = exponential_moment_integrand(Real.1, t)
        exponential_moment_integrand(Real.1, t) =
            t * exp_rate_density(Real.1, t)
        exp_rate_density(Real.1, t) = Real.1 * exp_decay(Real.1, t)
        exp_decay(Real.1, t) = exp_neg_fn(t)
        exp_rate_density(Real.1, t) = exp_neg_fn(t)
        exponential_moment_integrand(Real.1, t) = t * exp_neg_fn(t)
        gamma_fn(Nat.1, t) = exponential_moment_integrand(Real.1, t)
    }
    function_extensionality(gamma_fn(Nat.1), exponential_moment_integrand(Real.1))
    gamma_fn(Nat.1) = exponential_moment_integrand(Real.1)
}

/// The k = 1 Gamma integrand is bounded between zero and b on [0, b].
theorem gamma_fn_one_bounds_on(b: Real, t: Real) {
    Real.0 <= b and interval_contains(Real.0, b, t)
    implies (Real.0 <= gamma_fn(Nat.1, t) and gamma_fn(Nat.1, t) <= b)
} by {
    if Real.0 <= b and interval_contains(Real.0, b, t) {
        zero_is_smaller_than_one[Real]
        Real.0 < Real.1
        gamma_fn_one_eq_moment
        gamma_fn(Nat.1) = exponential_moment_integrand(Real.1)
        gamma_fn(Nat.1, t) = exponential_moment_integrand(Real.1, t)
        interval_contains_left(Real.0, b, t)
        Real.0 <= t
        exponential_moment_lower_bound_on(Real.1, b, t)
        Real.0 <= exponential_moment_integrand(Real.1, t)
        Real.0 <= gamma_fn(Nat.1, t)
        exponential_moment_upper_bound_on(Real.1, b, t)
        exponential_moment_integrand(Real.1, t) <= b * Real.1
        b * Real.1 = b
        exponential_moment_integrand(Real.1, t) <= b
        gamma_fn(Nat.1, t) <= b
        Real.0 <= gamma_fn(Nat.1, t) and gamma_fn(Nat.1, t) <= b
    }
}

/// The k = 1 Gamma integrand is integrable on [0, b] and its integral is
/// 1 - (b + 1)·e^(-b).
theorem gamma_interval_one(b: Real) {
    Real.0 <= b implies
    (is_integrable(gamma_fn(Nat.1), Real.0, b) and
     gamma_interval(Nat.1, Real.0, b) =
         Real.1 - exp_neg_fn(b) * (b + Real.1))
} by {
    if Real.0 <= b {
        zero_is_smaller_than_one[Real]
        Real.0 < Real.1
        exponential_moment_integrable(Real.1, b)
        is_integrable(exponential_moment_integrand(Real.1), Real.0, b)
        gamma_fn_one_eq_moment
        gamma_fn(Nat.1) = exponential_moment_integrand(Real.1)
        forall(t: Real) {
            if interval_contains(Real.0, b, t) {
                gamma_fn_one_eq_moment
                gamma_fn(Nat.1) = exponential_moment_integrand(Real.1)
                gamma_fn(Nat.1, t) = exponential_moment_integrand(Real.1, t)
            }
        }
        forall(t: Real) {
            if interval_contains(Real.0, b, t) {
                gamma_fn_one_bounds_on(b, t)
                Real.0 <= gamma_fn(Nat.1, t) and gamma_fn(Nat.1, t) <= b
                Real.0 <= gamma_fn(Nat.1, t)
            }
        }
        forall(t: Real) {
            if interval_contains(Real.0, b, t) {
                gamma_fn_one_bounds_on(b, t)
                Real.0 <= gamma_fn(Nat.1, t) and gamma_fn(Nat.1, t) <= b
                gamma_fn(Nat.1, t) <= b
            }
        }
        integral_and_integrable_eq_on(gamma_fn(Nat.1),
            exponential_moment_integrand(Real.1), Real.0, b, Real.0, b)
        is_integrable(gamma_fn(Nat.1), Real.0, b) and
            integral(gamma_fn(Nat.1), Real.0, b) =
                integral(exponential_moment_integrand(Real.1), Real.0, b)
        is_integrable(gamma_fn(Nat.1), Real.0, b)
        integral(gamma_fn(Nat.1), Real.0, b) =
            integral(exponential_moment_integrand(Real.1), Real.0, b)
        gamma_interval(Nat.1, Real.0, b) = integral(gamma_fn(Nat.1), Real.0, b)
        gamma_interval(Nat.1, Real.0, b) =
            integral(exponential_moment_integrand(Real.1), Real.0, b)
        exponential_mean_interval(Real.1, b)
        integral(exponential_moment_integrand(Real.1), Real.0, b) =
            Real.1 / Real.1 - exp_decay(Real.1, b) * (b + Real.1 / Real.1)
        Real.1 / Real.1 = Real.1
        Real.1 / Real.1 - exp_decay(Real.1, b) * (b + Real.1 / Real.1) =
            Real.1 - exp_decay(Real.1, b) * (b + Real.1)
        exp_decay_one_eq_exp_neg
        exp_decay(Real.1) = exp_neg_fn
        exp_decay(Real.1, b) = exp_neg_fn(b)
        Real.1 - exp_decay(Real.1, b) * (b + Real.1) =
            Real.1 - exp_neg_fn(b) * (b + Real.1)
        gamma_interval(Nat.1, Real.0, b) =
            Real.1 - exp_neg_fn(b) * (b + Real.1)
        is_integrable(gamma_fn(Nat.1), Real.0, b) and
            gamma_interval(Nat.1, Real.0, b) =
                Real.1 - exp_neg_fn(b) * (b + Real.1)
    }
}

// ---------------------------------------------------------------------------
// // The k = 2 case: ∫₀^b t²·e^(-t) dt = 2 - (b² + 2b + 2)·e^(-b)
// // ---------------------------------------------------------------------------

// /// The polynomial t² + 2t + 2.
// define erlang_two_poly(t: Real) -> Real {
//     t * t + two * t + two
// }

// /// The antiderivative of t²·e^(-t): t ↦ -(t² + 2t + 2)·e^(-t).
// define erlang_two_anti(t: Real) -> Real {
//     -erlang_two_poly(t) * exp_neg_fn(t)
// }

// /// The derivative of t²·e^(-t): t ↦ (2t - t²)·e^(-t).
// define erlang_two_deriv(t: Real) -> Real {
//     (two * t - t * t) * exp_neg_fn(t)
// }

// /// The square function t ↦ t·t.
// define sq_fn(t: Real) -> Real {
//     t * t
// }

// /// The square function is continuous.
// theorem sq_fn_continuous {
//     continuous(sq_fn)
// } by {
//     identity_function_is_continuous
//     continuous(identity_fn[Real])
//     continuous_pointwise_mul(identity_fn[Real], identity_fn[Real])
//     continuous(pointwise_mul(identity_fn[Real], identity_fn[Real]))
//     forall(x: Real) {
//         sq_fn(x) = x * x
//         pointwise_mul(identity_fn[Real], identity_fn[Real], x) =
//             identity_fn[Real](x) * identity_fn[Real](x)
//         identity_fn[Real](x) = x
//         pointwise_mul(identity_fn[Real], identity_fn[Real], x) = x * x
//         sq_fn(x) = pointwise_mul(identity_fn[Real], identity_fn[Real], x)
//     }
//     function_extensionality(sq_fn, pointwise_mul(identity_fn[Real], identity_fn[Real]))
//     define sq_cont_pred(h: Real -> Real) -> Bool {
//         continuous(h)
//     }
//     sq_cont_pred(pointwise_mul(identity_fn[Real], identity_fn[Real]))
//     function_eq_transport_predicate_rev(sq_cont_pred, sq_fn,
//         pointwise_mul(identity_fn[Real], identity_fn[Real]))
//     continuous(sq_fn)
// }

// /// The square function has derivative t ↦ 2t.
// theorem sq_fn_derivative {
//     is_derivative_fn(sq_fn, const_mul_left(two, identity_fn[Real]))
// } by {
//     derivative_fn_identity
//     is_derivative_fn(identity_fn[Real], constant[Real, Real](Real.1))
//     derivative_fn_square(identity_fn[Real], constant[Real, Real](Real.1))
//     is_derivative_fn(pointwise_mul(identity_fn[Real], identity_fn[Real]),
//         pointwise_add(pointwise_mul(identity_fn[Real], constant[Real, Real](Real.1)),
//             pointwise_mul(identity_fn[Real], constant[Real, Real](Real.1))))
//     forall(x: Real) {
//         pointwise_add(pointwise_mul(identity_fn[Real], constant[Real, Real](Real.1)),
//             pointwise_mul(identity_fn[Real], constant[Real, Real](Real.1)), x) =
//             pointwise_mul(identity_fn[Real], constant[Real, Real](Real.1), x) +
//             pointwise_mul(identity_fn[Real], constant[Real, Real](Real.1), x)
//         pointwise_mul(identity_fn[Real], constant[Real, Real](Real.1), x) =
//             identity_fn[Real](x) * constant[Real, Real](Real.1, x)
//         identity_fn[Real](x) = x
//         constant[Real, Real](Real.1, x) = Real.1
//         pointwise_mul(identity_fn[Real], constant[Real, Real](Real.1), x) = x
//         pointwise_add(pointwise_mul(identity_fn[Real], constant[Real, Real](Real.1)),
//             pointwise_mul(identity_fn[Real], constant[Real, Real](Real.1)), x) = x + x
//         const_mul_left(two, identity_fn[Real], x) = two * identity_fn[Real](x)
//         identity_fn[Real](x) = x
//         const_mul_left(two, identity_fn[Real], x) = two * x
//         x + x = two * x
//         pointwise_add(pointwise_mul(identity_fn[Real], constant[Real, Real](Real.1)),
//             pointwise_mul(identity_fn[Real], constant[Real, Real](Real.1)), x) =
//             const_mul_left(two, identity_fn[Real], x)
//     }
//     function_extensionality(
//         pointwise_add(pointwise_mul(identity_fn[Real], constant[Real, Real](Real.1)),
//             pointwise_mul(identity_fn[Real], constant[Real, Real](Real.1))),
//         const_mul_left(two, identity_fn[Real]))
//     define sq_deriv_pred(h: Real -> Real) -> Bool {
//         is_derivative_fn(pointwise_mul(identity_fn[Real], identity_fn[Real]), h)
//     }
//     sq_deriv_pred(pointwise_add(pointwise_mul(identity_fn[Real], constant[Real, Real](Real.1)),
//         pointwise_mul(identity_fn[Real], constant[Real, Real](Real.1))))
//     function_eq_transport_predicate_rev(sq_deriv_pred, const_mul_left(two, identity_fn[Real]),
//         pointwise_add(pointwise_mul(identity_fn[Real], constant[Real, Real](Real.1)),
//             pointwise_mul(identity_fn[Real], constant[Real, Real](Real.1))))
//     is_derivative_fn(pointwise_mul(identity_fn[Real], identity_fn[Real]),
//         const_mul_left(two, identity_fn[Real]))
//     forall(x: Real) {
//         sq_fn(x) = x * x
//         pointwise_mul(identity_fn[Real], identity_fn[Real], x) =
//             identity_fn[Real](x) * identity_fn[Real](x)
//         identity_fn[Real](x) = x
//         pointwise_mul(identity_fn[Real], identity_fn[Real], x) = x * x
//         sq_fn(x) = pointwise_mul(identity_fn[Real], identity_fn[Real], x)
//     }
//     function_extensionality(sq_fn, pointwise_mul(identity_fn[Real], identity_fn[Real]))
//     define sq_fn_pred(h: Real -> Real) -> Bool {
//         is_derivative_fn(h, const_mul_left(two, identity_fn[Real]))
//     }
//     sq_fn_pred(pointwise_mul(identity_fn[Real], identity_fn[Real]))
//     function_eq_transport_predicate_rev(sq_fn_pred, sq_fn,
//         pointwise_mul(identity_fn[Real], identity_fn[Real]))
//     is_derivative_fn(sq_fn, const_mul_left(two, identity_fn[Real]))
// }

// /// The polynomial t² + 2t + 2 is continuous.
// theorem erlang_two_poly_continuous {
//     continuous(erlang_two_poly)
// } by {
//     sq_fn_continuous
//     continuous(sq_fn)
//     identity_function_is_continuous
//     continuous(identity_fn[Real])
//     continuous_const_mul_left(two, identity_fn[Real])
//     continuous(const_mul_left(two, identity_fn[Real]))
//     continuous_pointwise_add(sq_fn, const_mul_left(two, identity_fn[Real]))
//     continuous(pointwise_add(sq_fn, const_mul_left(two, identity_fn[Real])))
//     constant_function_is_continuous(two)
//     continuous(constant[Real, Real](two))
//     continuous_pointwise_add(
//         pointwise_add(sq_fn, const_mul_left(two, identity_fn[Real])),
//         constant[Real, Real](two))
//     continuous(pointwise_add(
//         pointwise_add(sq_fn, const_mul_left(two, identity_fn[Real])),
//         constant[Real, Real](two)))
//     forall(x: Real) {
//         erlang_two_poly(x) = x * x + two * x + two
//         sq_fn(x) = x * x
//         const_mul_left(two, identity_fn[Real], x) = two * x
//         pointwise_add(sq_fn, const_mul_left(two, identity_fn[Real]), x) =
//             sq_fn(x) + const_mul_left(two, identity_fn[Real], x)
//         pointwise_add(sq_fn, const_mul_left(two, identity_fn[Real]), x) = x * x + two * x
//         constant[Real, Real](two, x) = two
//         pointwise_add(
//             pointwise_add(sq_fn, const_mul_left(two, identity_fn[Real])),
//             constant[Real, Real](two), x) =
//             pointwise_add(sq_fn, const_mul_left(two, identity_fn[Real]), x) +
//                 constant[Real, Real](two, x)
//         pointwise_add(
//             pointwise_add(sq_fn, const_mul_left(two, identity_fn[Real])),
//             constant[Real, Real](two), x) = x * x + two * x + two
//         erlang_two_poly(x) =
//             pointwise_add(
//                 pointwise_add(sq_fn, const_mul_left(two, identity_fn[Real])),
//                 constant[Real, Real](two), x)
//     }
//     function_extensionality(erlang_two_poly,
//         pointwise_add(
//             pointwise_add(sq_fn, const_mul_left(two, identity_fn[Real])),
//             constant[Real, Real](two)))
//     define poly_cont_pred(h: Real -> Real) -> Bool {
//         continuous(h)
//     }
//     poly_cont_pred(pointwise_add(
//         pointwise_add(sq_fn, const_mul_left(two, identity_fn[Real])),
//         constant[Real, Real](two)))
//     function_eq_transport_predicate_rev(poly_cont_pred, erlang_two_poly,
//         pointwise_add(
//             pointwise_add(sq_fn, const_mul_left(two, identity_fn[Real])),
//             constant[Real, Real](two)))
//     continuous(erlang_two_poly)
// }

// /// The polynomial t² + 2t + 2 has derivative t ↦ 2t + 2.
// theorem erlang_two_poly_derivative {
//     is_derivative_fn(erlang_two_poly,
//         pointwise_add(const_mul_left(two, identity_fn[Real]),
//             constant[Real, Real](two)))
// } by {
//     sq_fn_derivative
//     is_derivative_fn(sq_fn, const_mul_left(two, identity_fn[Real]))
//     derivative_fn_const_mul(two, identity_fn[Real], constant[Real, Real](Real.1))
//     is_derivative_fn(const_mul_left(two, identity_fn[Real]),
//         const_mul_left(two, constant[Real, Real](Real.1)))
//     derivative_fn_identity
//     is_derivative_fn(identity_fn[Real], constant[Real, Real](Real.1))
//     derivative_fn_constant(two)
//     is_derivative_fn(constant[Real, Real](two), constant[Real, Real](Real.0))
//     derivative_fn_add(sq_fn, const_mul_left(two, identity_fn[Real]),
//         const_mul_left(two, identity_fn[Real]),
//         const_mul_left(two, constant[Real, Real](Real.1)))
//     is_derivative_fn(pointwise_add(sq_fn, const_mul_left(two, identity_fn[Real])),
//         pointwise_add(const_mul_left(two, identity_fn[Real]),
//             const_mul_left(two, constant[Real, Real](Real.1))))
//     forall(x: Real) {
//         const_mul_left(two, constant[Real, Real](Real.1), x) =
//             two * constant[Real, Real](Real.1, x)
//         constant[Real, Real](Real.1, x) = Real.1
//         const_mul_left(two, constant[Real, Real](Real.1), x) = two
//         pointwise_add(const_mul_left(two, identity_fn[Real]),
//             const_mul_left(two, constant[Real, Real](Real.1)), x) =
//             const_mul_left(two, identity_fn[Real], x) +
//                 const_mul_left(two, constant[Real, Real](Real.1), x)
//         const_mul_left(two, identity_fn[Real], x) = two * identity_fn[Real](x)
//         identity_fn[Real](x) = x
//         const_mul_left(two, identity_fn[Real], x) = two * x
//         pointwise_add(const_mul_left(two, identity_fn[Real]),
//             const_mul_left(two, constant[Real, Real](Real.1)), x) = two * x + two
//         constant[Real, Real](two, x) = two
//         pointwise_add(const_mul_left(two, identity_fn[Real]),
//             constant[Real, Real](two), x) =
//             const_mul_left(two, identity_fn[Real], x) + constant[Real, Real](two, x)
//         pointwise_add(const_mul_left(two, identity_fn[Real]),
//             constant[Real, Real](two), x) = two * x + two
//         pointwise_add(const_mul_left(two, identity_fn[Real]),
//             const_mul_left(two, constant[Real, Real](Real.1)), x) =
//             pointwise_add(const_mul_left(two, identity_fn[Real]),
//                 constant[Real, Real](two), x)
//     }
//     function_extensionality(
//         pointwise_add(const_mul_left(two, identity_fn[Real]),
//             const_mul_left(two, constant[Real, Real](Real.1))),
//         pointwise_add(const_mul_left(two, identity_fn[Real]),
//             constant[Real, Real](two)))
//     derivative_fn_add(
//         pointwise_add(sq_fn, const_mul_left(two, identity_fn[Real])),
//         constant[Real, Real](two),
//         pointwise_add(const_mul_left(two, identity_fn[Real]),
//             constant[Real, Real](two)),
//         constant[Real, Real](Real.0))
//     is_derivative_fn(
//         pointwise_add(
//             pointwise_add(sq_fn, const_mul_left(two, identity_fn[Real])),
//             constant[Real, Real](two)),
//         pointwise_add(
//             pointwise_add(const_mul_left(two, identity_fn[Real]),
//                 constant[Real, Real](two)),
//             constant[Real, Real](Real.0)))
//     forall(x: Real) {
//         pointwise_add(
//             pointwise_add(const_mul_left(two, identity_fn[Real]),
//                 constant[Real, Real](two)),
//             constant[Real, Real](Real.0), x) =
//             pointwise_add(const_mul_left(two, identity_fn[Real]),
//                 constant[Real, Real](two), x) + constant[Real, Real](Real.0, x)
//         constant[Real, Real](Real.0, x) = Real.0
//         pointwise_add(const_mul_left(two, identity_fn[Real]),
//             constant[Real, Real](two), x) + Real.0 =
//             pointwise_add(const_mul_left(two, identity_fn[Real]),
//                 constant[Real, Real](two), x)
//         pointwise_add(
//             pointwise_add(const_mul_left(two, identity_fn[Real]),
//                 constant[Real, Real](two)),
//             constant[Real, Real](Real.0), x) =
//             pointwise_add(const_mul_left(two, identity_fn[Real]),
//                 constant[Real, Real](two), x)
//     }
//     function_extensionality(
//         pointwise_add(
//             pointwise_add(const_mul_left(two, identity_fn[Real]),
//                 constant[Real, Real](two)),
//             constant[Real, Real](Real.0)),
//         pointwise_add(const_mul_left(two, identity_fn[Real]),
//             constant[Real, Real](two)))
//     forall(x: Real) {
//         erlang_two_poly(x) = x * x + two * x + two
//         sq_fn(x) = x * x
//         const_mul_left(two, identity_fn[Real], x) = two * x
//         pointwise_add(sq_fn, const_mul_left(two, identity_fn[Real]), x) =
//             sq_fn(x) + const_mul_left(two, identity_fn[Real], x)
//         pointwise_add(sq_fn, const_mul_left(two, identity_fn[Real]), x) = x * x + two * x
//         constant[Real, Real](two, x) = two
//         pointwise_add(
//             pointwise_add(sq_fn, const_mul_left(two, identity_fn[Real])),
//             constant[Real, Real](two), x) =
//             pointwise_add(sq_fn, const_mul_left(two, identity_fn[Real]), x) +
//                 constant[Real, Real](two, x)
//         pointwise_add(
//             pointwise_add(sq_fn, const_mul_left(two, identity_fn[Real])),
//             constant[Real, Real](two), x) = x * x + two * x + two
//         erlang_two_poly(x) =
//             pointwise_add(
//                 pointwise_add(sq_fn, const_mul_left(two, identity_fn[Real])),
//                 constant[Real, Real](two), x)
//     }
//     function_extensionality(erlang_two_poly,
//         pointwise_add(
//             pointwise_add(sq_fn, const_mul_left(two, identity_fn[Real])),
//             constant[Real, Real](two)))
//     is_derivative_fn_both_eq(
//         pointwise_add(
//             pointwise_add(sq_fn, const_mul_left(two, identity_fn[Real])),
//             constant[Real, Real](two)),
//         erlang_two_poly,
//         pointwise_add(
//             pointwise_add(const_mul_left(two, identity_fn[Real]),
//                 constant[Real, Real](two)),
//             constant[Real, Real](Real.0)),
//         pointwise_add(const_mul_left(two, identity_fn[Real]),
//             constant[Real, Real](two)))
//     is_derivative_fn(erlang_two_poly,
//         pointwise_add(const_mul_left(two, identity_fn[Real]),
//             constant[Real, Real](two)))
// }

// /// The antiderivative of t²·e^(-t).
// theorem erlang_two_anti_derivative {
//     is_derivative_fn(erlang_two_anti, gamma_fn(Nat.2))
// } by {
//     erlang_two_poly_derivative
//     is_derivative_fn(erlang_two_poly,
//         pointwise_add(const_mul_left(two, identity_fn[Real]),
//             constant[Real, Real](two)))
//     exp_neg_fn_derivative
//     is_derivative_fn(exp_neg_fn, pointwise_neg(exp_neg_fn))
//     derivative_fn_mul(erlang_two_poly, exp_neg_fn,
//         pointwise_add(const_mul_left(two, identity_fn[Real]),
//             constant[Real, Real](two)),
//         pointwise_neg(exp_neg_fn))
//     is_derivative_fn(
//         pointwise_mul(erlang_two_poly, exp_neg_fn),
//         pointwise_add(
//             pointwise_mul(erlang_two_poly, pointwise_neg(exp_neg_fn)),
//             pointwise_mul(exp_neg_fn,
//                 pointwise_add(const_mul_left(two, identity_fn[Real]),
//                     constant[Real, Real](two)))))
//     is_derivative_fn_pointwise_add_comm(
//         pointwise_mul(erlang_two_poly, exp_neg_fn),
//         pointwise_mul(erlang_two_poly, pointwise_neg(exp_neg_fn)),
//         pointwise_mul(exp_neg_fn,
//             pointwise_add(const_mul_left(two, identity_fn[Real]),
//                 constant[Real, Real](two))))
//     is_derivative_fn(
//         pointwise_mul(erlang_two_poly, exp_neg_fn),
//         pointwise_add(
//             pointwise_mul(pointwise_add(const_mul_left(two, identity_fn[Real]),
//                 constant[Real, Real](two)), exp_neg_fn),
//             pointwise_mul(erlang_two_poly, pointwise_neg(exp_neg_fn))))
//     forall(x: Real) {
//         pointwise_mul(pointwise_add(const_mul_left(two, identity_fn[Real]),
//             constant[Real, Real](two)), exp_neg_fn, x) =
//             pointwise_add(const_mul_left(two, identity_fn[Real]),
//                 constant[Real, Real](two), x) * exp_neg_fn(x)
//         pointwise_add(const_mul_left(two, identity_fn[Real]),
//             constant[Real, Real](two), x) =
//             const_mul_left(two, identity_fn[Real], x) + constant[Real, Real](two, x)
//         const_mul_left(two, identity_fn[Real], x) = two * x
//         constant[Real, Real](two, x) = two
//         pointwise_add(const_mul_left(two, identity_fn[Real]),
//             constant[Real, Real](two), x) = two * x + two
//         pointwise_mul(pointwise_add(const_mul_left(two, identity_fn[Real]),
//             constant[Real, Real](two)), exp_neg_fn, x) = (two * x + two) * exp_neg_fn(x)
//         pointwise_mul(erlang_two_poly, pointwise_neg(exp_neg_fn), x) =
//             erlang_two_poly(x) * pointwise_neg(exp_neg_fn, x)
//         erlang_two_poly(x) = x * x + two * x + two
//         pointwise_neg(exp_neg_fn, x) = -exp_neg_fn(x)
//         pointwise_mul(erlang_two_poly, pointwise_neg(exp_neg_fn), x) =
//             (x * x + two * x + two) * (-exp_neg_fn(x))
//         (x * x + two * x + two) * (-exp_neg_fn(x)) =
//             -((x * x + two * x + two) * exp_neg_fn(x))
//         pointwise_mul(erlang_two_poly, pointwise_neg(exp_neg_fn), x) =
//             -(x * x + two * x + two) * exp_neg_fn(x)
//         pointwise_add(
//             pointwise_mul(pointwise_add(const_mul_left(two, identity_fn[Real]),
//                 constant[Real, Real](two)), exp_neg_fn),
//             pointwise_mul(erlang_two_poly, pointwise_neg(exp_neg_fn)), x) =
//             pointwise_mul(pointwise_add(const_mul_left(two, identity_fn[Real]),
//                 constant[Real, Real](two)), exp_neg_fn, x) +
//             pointwise_mul(erlang_two_poly, pointwise_neg(exp_neg_fn), x)
//         pointwise_mul(pointwise_add(const_mul_left(two, identity_fn[Real]),
//             constant[Real, Real](two)), exp_neg_fn, x) +
//             pointwise_mul(erlang_two_poly, pointwise_neg(exp_neg_fn), x) =
//             (two * x + two) * exp_neg_fn(x) - (x * x + two * x + two) * exp_neg_fn(x)
//         pointwise_add(
//             pointwise_mul(pointwise_add(const_mul_left(two, identity_fn[Real]),
//                 constant[Real, Real](two)), exp_neg_fn),
//             pointwise_mul(erlang_two_poly, pointwise_neg(exp_neg_fn)), x) =
//             (two * x + two) * exp_neg_fn(x) - (x * x + two * x + two) * exp_neg_fn(x)
//         (two * x + two) * exp_neg_fn(x) - (x * x + two * x + two) * exp_neg_fn(x) =
//             (two * x + two - (x * x + two * x + two)) * exp_neg_fn(x)
//         two = Real.1 + Real.1
//         two * x = (Real.1 + Real.1) * x
//         (Real.1 + Real.1) * x = Real.1 * x + Real.1 * x
//         Real.1 * x = x
//         Real.1 * x + Real.1 * x = x + x
//         two * x = x + x
//         two * x + two = x + x + (Real.1 + Real.1)
//         x + x + (Real.1 + Real.1) - (x * x + x + x + (Real.1 + Real.1)) = -(x * x)
//         two * x + two - (x * x + two * x + two) = -(x * x)
//         (two * x + two - (x * x + two * x + two)) * exp_neg_fn(x) = -(x * x) * exp_neg_fn(x)
//         (two * x + two) * exp_neg_fn(x) - (x * x + two * x + two) * exp_neg_fn(x) =
//             -(x * x) * exp_neg_fn(x)
//         // -(t²)·e^(-t) = -(t·t)·e^(-t)
//         x * x = x.pow(Nat.2)
//         gamma_fn(Nat.2, x) = pow_real_fn(Nat.2, x) * exp_neg_fn(x)
//         pow_real_fn(Nat.2, x) = x.pow(Nat.2)
//         gamma_fn(Nat.2, x) = x.pow(Nat.2) * exp_neg_fn(x)
//         -(x * x) * exp_neg_fn(x) = -(gamma_fn(Nat.2, x))
//         pointwise_add(
//             pointwise_mul(pointwise_add(const_mul_left(two, identity_fn[Real]),
//                 constant[Real, Real](two)), exp_neg_fn),
//             pointwise_mul(erlang_two_poly, pointwise_neg(exp_neg_fn)), x) =
//             -gamma_fn(Nat.2, x)
//         pointwise_neg(gamma_fn(Nat.2), x) = -gamma_fn(Nat.2, x)
//         pointwise_add(
//             pointwise_mul(pointwise_add(const_mul_left(two, identity_fn[Real]),
//                 constant[Real, Real](two)), exp_neg_fn),
//             pointwise_mul(erlang_two_poly, pointwise_neg(exp_neg_fn)), x) =
//             pointwise_neg(gamma_fn(Nat.2), x)
//     }
//     function_extensionality(
//         pointwise_add(
//             pointwise_mul(pointwise_add(const_mul_left(two, identity_fn[Real]),
//                 constant[Real, Real](two)), exp_neg_fn),
//             pointwise_mul(erlang_two_poly, pointwise_neg(exp_neg_fn))),
//         pointwise_neg(gamma_fn(Nat.2)))
//     derivative_fn_neg(
//         pointwise_mul(erlang_two_poly, exp_neg_fn),
//         pointwise_neg(gamma_fn(Nat.2)))
//     is_derivative_fn(
//         pointwise_neg(pointwise_mul(erlang_two_poly, exp_neg_fn)),
//         pointwise_neg(pointwise_neg(gamma_fn(Nat.2))))
//     forall(x: Real) {
//         erlang_two_anti(x) = -erlang_two_poly(x) * exp_neg_fn(x)
//         pointwise_mul(erlang_two_poly, exp_neg_fn, x) =
//             erlang_two_poly(x) * exp_neg_fn(x)
//         pointwise_neg(pointwise_mul(erlang_two_poly, exp_neg_fn), x) =
//             -pointwise_mul(erlang_two_poly, exp_neg_fn, x)
//         pointwise_neg(pointwise_mul(erlang_two_poly, exp_neg_fn), x) =
//             -erlang_two_poly(x) * exp_neg_fn(x)
//         erlang_two_anti(x) =
//             pointwise_neg(pointwise_mul(erlang_two_poly, exp_neg_fn), x)
//         pointwise_neg(pointwise_neg(gamma_fn(Nat.2)), x) =
//             -pointwise_neg(gamma_fn(Nat.2), x)
//         pointwise_neg(gamma_fn(Nat.2), x) = -gamma_fn(Nat.2, x)
//         -(-gamma_fn(Nat.2, x)) = gamma_fn(Nat.2, x)
//         pointwise_neg(pointwise_neg(gamma_fn(Nat.2)), x) = gamma_fn(Nat.2, x)
//     }
//     function_extensionality(
//         pointwise_neg(pointwise_neg(gamma_fn(Nat.2))), gamma_fn(Nat.2))
//     eq_true_intro(is_derivative_fn(
//         pointwise_neg(pointwise_mul(erlang_two_poly, exp_neg_fn)),
//         pointwise_neg(pointwise_neg(gamma_fn(Nat.2)))))
//     (is_derivative_fn(
//         pointwise_neg(pointwise_mul(erlang_two_poly, exp_neg_fn)),
//         pointwise_neg(pointwise_neg(gamma_fn(Nat.2))))) = true
//     function_eq_transport_predicate_rev[Real, Real](
//         function(h: Real -> Real) {
//             is_derivative_fn(pointwise_neg(pointwise_mul(erlang_two_poly, exp_neg_fn)), h)
//         },
//         pointwise_neg(pointwise_neg(gamma_fn(Nat.2))), gamma_fn(Nat.2))
//     is_derivative_fn(
//         pointwise_neg(pointwise_mul(erlang_two_poly, exp_neg_fn)), gamma_fn(Nat.2))
//     function_extensionality(
//         pointwise_neg(pointwise_mul(erlang_two_poly, exp_neg_fn)),
//         erlang_two_anti)
//     erlang_two_anti = pointwise_neg(pointwise_mul(erlang_two_poly, exp_neg_fn))
//     eq_true_intro(is_derivative_fn(
//         pointwise_neg(pointwise_mul(erlang_two_poly, exp_neg_fn)), gamma_fn(Nat.2)))
//     (is_derivative_fn(
//         pointwise_neg(pointwise_mul(erlang_two_poly, exp_neg_fn)), gamma_fn(Nat.2))) = true
//     function_eq_transport_predicate_rev[Real, Real](
//         function(h: Real -> Real) { is_derivative_fn(h, gamma_fn(Nat.2)) },
//         erlang_two_anti, pointwise_neg(pointwise_mul(erlang_two_poly, exp_neg_fn)))
//     is_derivative_fn(erlang_two_anti, gamma_fn(Nat.2))
// }

// /// The derivative of t²·e^(-t).
// theorem erlang_two_derivative {
//     is_derivative_fn(gamma_fn(Nat.2), erlang_two_deriv)
// } by {
//     erlang_two_poly_derivative
//     is_derivative_fn(erlang_two_poly,
//         pointwise_add(const_mul_left(two, identity_fn[Real]),
//             constant[Real, Real](two)))
//     exp_neg_fn_derivative
//     is_derivative_fn(exp_neg_fn, pointwise_neg(exp_neg_fn))
//     derivative_fn_mul(erlang_two_poly, exp_neg_fn,
//         pointwise_add(const_mul_left(two, identity_fn[Real]),
//             constant[Real, Real](two)),
//         pointwise_neg(exp_neg_fn))
//     is_derivative_fn(
//         pointwise_mul(erlang_two_poly, exp_neg_fn),
//         pointwise_add(
//             pointwise_mul(erlang_two_poly, pointwise_neg(exp_neg_fn)),
//             pointwise_mul(exp_neg_fn,
//                 pointwise_add(const_mul_left(two, identity_fn[Real]),
//                     constant[Real, Real](two)))))
//     is_derivative_fn_pointwise_add_comm(
//         pointwise_mul(erlang_two_poly, exp_neg_fn),
//         pointwise_mul(erlang_two_poly, pointwise_neg(exp_neg_fn)),
//         pointwise_mul(exp_neg_fn,
//             pointwise_add(const_mul_left(two, identity_fn[Real]),
//                 constant[Real, Real](two))))
//     is_derivative_fn(
//         pointwise_mul(erlang_two_poly, exp_neg_fn),
//         pointwise_add(
//             pointwise_mul(pointwise_add(const_mul_left(two, identity_fn[Real]),
//                 constant[Real, Real](two)), exp_neg_fn),
//             pointwise_mul(erlang_two_poly, pointwise_neg(exp_neg_fn))))
//     forall(x: Real) {
//         gamma_fn(Nat.2, x) = pow_real_fn(Nat.2, x) * exp_neg_fn(x)
//         pow_real_fn(Nat.2, x) = x.pow(Nat.2)
//         x.pow(Nat.2) = x * x
//         pow_real_fn(Nat.2, x) = x * x
//         gamma_fn(Nat.2, x) = (x * x) * exp_neg_fn(x)
//         pointwise_mul(erlang_two_poly, exp_neg_fn, x) =
//             erlang_two_poly(x) * exp_neg_fn(x)
//         erlang_two_poly(x) = x * x + two * x + two
//         pointwise_mul(erlang_two_poly, exp_neg_fn, x) =
//             (x * x + two * x + two) * exp_neg_fn(x)
//         two = Real.1 + Real.1
//         two * x = (Real.1 + Real.1) * x
//         (Real.1 + Real.1) * x = Real.1 * x + Real.1 * x
//         Real.1 * x = x
//         Real.1 * x + Real.1 * x = x + x
//         two * x = x + x
//         x * x + two * x + two = x * x + (x + x) + (Real.1 + Real.1)
//         (x * x + (x + x) + (Real.1 + Real.1)) * exp_neg_fn(x) = (x * x) * exp_neg_fn(x)
//         x * x + two * x + two = x * x + (x + x) + (Real.1 + Real.1)
//         (x * x + two * x + two) * exp_neg_fn(x) = (x * x) * exp_neg_fn(x)
//         gamma_fn(Nat.2, x) = pointwise_mul(erlang_two_poly, exp_neg_fn, x)
//         pointwise_add(
//             pointwise_mul(pointwise_add(const_mul_left(two, identity_fn[Real]),
//                 constant[Real, Real](two)), exp_neg_fn),
//             pointwise_mul(erlang_two_poly, pointwise_neg(exp_neg_fn)), x) =
//             pointwise_mul(pointwise_add(const_mul_left(two, identity_fn[Real]),
//                 constant[Real, Real](two)), exp_neg_fn, x) +
//             pointwise_mul(erlang_two_poly, pointwise_neg(exp_neg_fn), x)
//         pointwise_mul(pointwise_add(const_mul_left(two, identity_fn[Real]),
//             constant[Real, Real](two)), exp_neg_fn, x) +
//             pointwise_mul(erlang_two_poly, pointwise_neg(exp_neg_fn), x) =
//             (two * x + two) * exp_neg_fn(x) - (x * x + two * x + two) * exp_neg_fn(x)
//         pointwise_add(
//             pointwise_mul(pointwise_add(const_mul_left(two, identity_fn[Real]),
//                 constant[Real, Real](two)), exp_neg_fn),
//             pointwise_mul(erlang_two_poly, pointwise_neg(exp_neg_fn)), x) =
//             (two * x + two) * exp_neg_fn(x) - (x * x + two * x + two) * exp_neg_fn(x)
//         (two * x + two) * exp_neg_fn(x) - (x * x + two * x + two) * exp_neg_fn(x) =
//             (two * x + two - (x * x + two * x + two)) * exp_neg_fn(x)
//         two = Real.1 + Real.1
//         two * x = x + x
//         x + x + (Real.1 + Real.1) - (x * x + x + x + (Real.1 + Real.1)) = -(x * x)
//         two * x + two - (x * x + two * x + two) = -(x * x)
//         x * x + two * x + two = (x * x) + (two * x + two)
//         -(x * x + two * x + two) = -(x * x) - (two * x + two)
//         (two * x + two) - (x * x + two * x + two) = (two * x + two) - (x * x) - (two * x + two)
//         (two * x + two) - (x * x) - (two * x + two) = -(x * x)
//         two * x + two - (x * x + two * x + two) = two * x - x * x
//         (two * x + two - (x * x + two * x + two)) * exp_neg_fn(x) =
//             (two * x - x * x) * exp_neg_fn(x)
//         (two * x + two) * exp_neg_fn(x) - (x * x + two * x + two) * exp_neg_fn(x) =
//             (two * x - x * x) * exp_neg_fn(x)
//         erlang_two_deriv(x) = (two * x - x * x) * exp_neg_fn(x)
//         pointwise_add(
//             pointwise_mul(pointwise_add(const_mul_left(two, identity_fn[Real]),
//                 constant[Real, Real](two)), exp_neg_fn),
//             pointwise_mul(erlang_two_poly, pointwise_neg(exp_neg_fn)), x) =
//             erlang_two_deriv(x)
//     }
//     function_extensionality(
//         pointwise_mul(erlang_two_poly, exp_neg_fn), gamma_fn(Nat.2))
//     function_extensionality(
//         pointwise_add(
//             pointwise_mul(pointwise_add(const_mul_left(two, identity_fn[Real]),
//                 constant[Real, Real](two)), exp_neg_fn),
//             pointwise_mul(erlang_two_poly, pointwise_neg(exp_neg_fn))),
//         erlang_two_deriv)
//     eq_true_intro(is_derivative_fn(
//         pointwise_mul(erlang_two_poly, exp_neg_fn),
//         pointwise_add(
//             pointwise_mul(pointwise_add(const_mul_left(two, identity_fn[Real]),
//                 constant[Real, Real](two)), exp_neg_fn),
//             pointwise_mul(erlang_two_poly, pointwise_neg(exp_neg_fn)))))
//     (is_derivative_fn(
//         pointwise_mul(erlang_two_poly, exp_neg_fn),
//         pointwise_add(
//             pointwise_mul(pointwise_add(const_mul_left(two, identity_fn[Real]),
//                 constant[Real, Real](two)), exp_neg_fn),
//             pointwise_mul(erlang_two_poly, pointwise_neg(exp_neg_fn))))) = true
//     function_eq_transport_predicate_rev[Real, Real](
//         function(h: Real -> Real) {
//             is_derivative_fn(pointwise_mul(erlang_two_poly, exp_neg_fn), h)
//         },
//         pointwise_add(
//             pointwise_mul(pointwise_add(const_mul_left(two, identity_fn[Real]),
//                 constant[Real, Real](two)), exp_neg_fn),
//             pointwise_mul(erlang_two_poly, pointwise_neg(exp_neg_fn))),
//         erlang_two_deriv)
//     is_derivative_fn(pointwise_mul(erlang_two_poly, exp_neg_fn), erlang_two_deriv)
//     eq_true_intro(is_derivative_fn(
//         pointwise_mul(erlang_two_poly, exp_neg_fn), erlang_two_deriv))
//     (is_derivative_fn(
//         pointwise_mul(erlang_two_poly, exp_neg_fn), erlang_two_deriv)) = true
//     function_eq_transport_predicate_rev[Real, Real](
//         function(h: Real -> Real) { is_derivative_fn(h, erlang_two_deriv) },
//         gamma_fn(Nat.2), pointwise_mul(erlang_two_poly, exp_neg_fn))
//     is_derivative_fn(gamma_fn(Nat.2), erlang_two_deriv)
// }

// /// t²·e^(-t) is bounded between zero and b² on [0, b].
// theorem gamma_fn_two_bounds_on(b: Real, t: Real) {
//     Real.0 <= b and interval_contains(Real.0, b, t)
//     implies (Real.0 <= gamma_fn(Nat.2, t) and gamma_fn(Nat.2, t) <= b * b)
// } by {
//     if Real.0 <= b and interval_contains(Real.0, b, t) {
//         gamma_fn_nonneg(Nat.2, t)
//         interval_contains_left(Real.0, b, t)
//         Real.0 <= t
//         Real.0 <= gamma_fn(Nat.2, t)
//         gamma_fn(Nat.2, t) = pow_real_fn(Nat.2, t) * exp_neg_fn(t)
//         pow_real_fn(Nat.2, t) = t.pow(Nat.2)
//         t.pow(Nat.2) = t * t
//         pow_real_fn(Nat.2, t) = t * t
//         gamma_fn(Nat.2, t) = (t * t) * exp_neg_fn(t)
//         interval_contains_left(Real.0, b, t)
//         Real.0 <= t
//         t <= b
//         interval_contains_right(Real.0, b, t)
//         mul_nonneg_lte(t, t)
//         Real.0 <= t * t
//         mul_le_mul_nonneg_lte(t, t, b, b)
//         t * t <= b * b
//         exp_neg_fn_le_one(t)
//         exp_neg_fn(t) <= Real.1
//         mul_le_mul_of_nonneg_right(t * t, b * b, exp_neg_fn(t))
//         (t * t) * exp_neg_fn(t) <= (b * b) * exp_neg_fn(t)
//         mul_le_mul_of_nonneg_right(exp_neg_fn(t), Real.1, b * b)
//         exp_neg_fn(t) * (b * b) <= Real.1 * (b * b)
//         real_mul_comm(b * b, exp_neg_fn(t))
//         exp_neg_fn(t) * (b * b) = (b * b) * exp_neg_fn(t)
//         Real.1 * (b * b) = b * b
//         (b * b) * exp_neg_fn(t) <= b * b
//         lte_trans((t * t) * exp_neg_fn(t), (b * b) * exp_neg_fn(t), b * b)
//         (t * t) * exp_neg_fn(t) <= b * b
//         gamma_fn(Nat.2, t) <= b * b
//         Real.0 <= gamma_fn(Nat.2, t) and gamma_fn(Nat.2, t) <= b * b
//     }
// }

// /// The derivative of t²·e^(-t) is bounded on [0, b].
// theorem erlang_two_deriv_bound(b: Real) {
//     Real.0 <= b implies
//     forall(z: Real) {
//         interval_contains(Real.0, b, z) implies erlang_two_deriv(z).abs <= two * b + b * b
//     }
// } by {
//     if Real.0 <= b {
//         forall(z: Real) {
//             if interval_contains(Real.0, b, z) {
//                 erlang_two_deriv(z) = (two * z - z * z) * exp_neg_fn(z)
//                 mul_abs(two * z - z * z, exp_neg_fn(z))
//                 ((two * z - z * z) * exp_neg_fn(z)).abs =
//                     (two * z - z * z).abs * exp_neg_fn(z).abs
//                 erlang_two_deriv(z).abs =
//                     (two * z - z * z).abs * exp_neg_fn(z).abs
//                 exp_neg_fn_nonneg(z)
//                 Real.0 <= exp_neg_fn(z)
//                 abs_of_nonneg(exp_neg_fn(z))
//                 exp_neg_fn(z).abs = exp_neg_fn(z)
//                 erlang_two_deriv(z).abs = (two * z - z * z).abs * exp_neg_fn(z)
//                 // |2z - z²| <= |2z| + |z²| = 2z + z² <= 2b + b²
//                 triangle_ineq(two * z, -(z * z))
//                 (two * z + -(z * z)).abs <= (two * z).abs + (-(z * z)).abs
//                 two * z - z * z = two * z + -(z * z)
//                 (two * z - z * z).abs <= (two * z).abs + (-(z * z)).abs
//                 interval_contains_left(Real.0, b, z)
//                 Real.0 <= z
//                 mul_nonneg_lte(two, z)
//                 Real.0 <= two * z
//                 abs_of_nonneg(two * z)
//                 (two * z).abs = two * z
//                 abs_neg(z * z)
//                 (-(z * z)).abs = (z * z).abs
//                 mul_nonneg(z, z)
//                 z * z >= Real.0
//                 abs_of_nonneg(z * z)
//                 (z * z).abs = z * z
//                 (-(z * z)).abs = z * z
//                 (two * z).abs + (-(z * z)).abs = two * z + z * z
//                 (two * z - z * z).abs <= two * z + z * z
//                 interval_contains_right(Real.0, b, z)
//                 z <= b
//                 two_positive
//                 two > Real.0
//                 lt_imp_lte(Real.0, two)
//                 Real.0 <= two
//                 mul_le_mul_of_nonneg_right(z, b, two)
//                 z * two <= b * two
//                 real_mul_comm(two, z)
//                 two * z = z * two
//                 real_mul_comm(two, b)
//                 two * b = b * two
//                 two * z <= two * b
//                 mul_le_mul_nonneg_lte(z, z, b, b)
//                 z * z <= b * b
//                 add_le_add(two * z, two * b, z * z, b * b)
//                 two * z + z * z <= two * b + b * b
//                 lte_trans((two * z - z * z).abs, two * z + z * z, two * b + b * b)
//                 (two * z - z * z).abs <= two * b + b * b
//                 // multiply by 0 <= e^(-z) <= 1
//                 abs_gte_zero(two * z - z * z)
//                 Real.0 <= (two * z - z * z).abs
//                 exp_neg_fn_le_one(z)
//                 exp_neg_fn(z) <= Real.1
//                 zero_is_smaller_than_one[Real]
//                 Real.0 < Real.1
//                 lt_imp_lte(Real.0, Real.1)
//                 Real.0 <= Real.1
//                 two_positive
//                 two > Real.0
//                 lt_imp_lte(Real.0, two)
//                 Real.0 <= two
//                 interval_contains_left(Real.0, b, b)
//                 Real.0 <= b
//                 mul_nonneg_lte(two, b)
//                 Real.0 <= two * b
//                 mul_nonneg(b, b)
//                 b * b >= Real.0
//                 Real.0 <= b * b
//                 add_le_add(Real.0, two * b, Real.0, b * b)
//                 Real.0 + Real.0 <= two * b + b * b
//                 Real.0 + Real.0 = Real.0
//                 Real.0 <= two * b + b * b
//                 mul_le_mul_nonneg_lte((two * z - z * z).abs, exp_neg_fn(z),
//                     two * b + b * b, Real.1)
//                 (two * z - z * z).abs * exp_neg_fn(z) <= (two * b + b * b) * Real.1
//                 (two * b + b * b) * Real.1 = two * b + b * b
//                 (two * z - z * z).abs * exp_neg_fn(z) <= two * b + b * b
//                 erlang_two_deriv(z).abs <= two * b + b * b
//             }
//         }
//     }
// }

// /// t²·e^(-t) is integrable on [0, b] and its integral is
// /// 2 - (b² + 2b + 2)·e^(-b).
// theorem gamma_interval_two(b: Real) {
//     Real.0 <= b implies
//     (is_integrable(gamma_fn(Nat.2), Real.0, b) and
//      gamma_interval(Nat.2, Real.0, b) =
//          two - erlang_two_poly(b) * exp_neg_fn(b))
// } by {
//     if Real.0 <= b {
//         erlang_two_deriv_bound(b)
//         forall(z: Real) {
//             if interval_contains(Real.0, b, z) {
//                 erlang_two_deriv(z).abs <= two * b + b * b
//             }
//         }
//         forall(t: Real) {
//             if interval_contains(Real.0, b, t) {
//                 gamma_fn_two_bounds_on(b, t)
//                 Real.0 <= gamma_fn(Nat.2, t) and gamma_fn(Nat.2, t) <= b * b
//                 Real.0 <= gamma_fn(Nat.2, t)
//             }
//         }
//         forall(t: Real) {
//             if interval_contains(Real.0, b, t) {
//                 gamma_fn_two_bounds_on(b, t)
//                 Real.0 <= gamma_fn(Nat.2, t) and gamma_fn(Nat.2, t) <= b * b
//                 gamma_fn(Nat.2, t) <= b * b
//             }
//         }
//         // 0 <= 2b + b²
//         two_positive
//         two > Real.0
//         lt_imp_lte(Real.0, two)
//         Real.0 <= two
//         interval_contains_left(Real.0, b, b)
//         Real.0 <= b
//         mul_nonneg_lte(two, b)
//         Real.0 <= two * b
//         mul_nonneg(b, b)
//         b * b >= Real.0
//         Real.0 <= b * b
//         add_le_add(Real.0, two * b, Real.0, b * b)
//         Real.0 + Real.0 <= two * b + b * b
//         Real.0 + Real.0 = Real.0
//         Real.0 <= two * b + b * b
//         erlang_two_anti_derivative
//         is_derivative_fn(erlang_two_anti, gamma_fn(Nat.2))
//         erlang_two_derivative
//         is_derivative_fn(gamma_fn(Nat.2), erlang_two_deriv)
//         sq_fn_continuous
//         continuous(sq_fn)
//         erlang_two_poly_continuous
//         continuous(erlang_two_poly)
//         exp_neg_fn_continuous
//         continuous(exp_neg_fn)
//         continuous_pointwise_mul(erlang_two_poly, exp_neg_fn)
//         continuous(pointwise_mul(erlang_two_poly, exp_neg_fn))
//         continuous_pointwise_neg(pointwise_mul(erlang_two_poly, exp_neg_fn))
//         continuous(pointwise_neg(pointwise_mul(erlang_two_poly, exp_neg_fn)))
//         forall(x: Real) {
//             erlang_two_anti(x) = -erlang_two_poly(x) * exp_neg_fn(x)
//             pointwise_mul(erlang_two_poly, exp_neg_fn, x) =
//                 erlang_two_poly(x) * exp_neg_fn(x)
//             pointwise_neg(pointwise_mul(erlang_two_poly, exp_neg_fn), x) =
//                 -pointwise_mul(erlang_two_poly, exp_neg_fn, x)
//             pointwise_neg(pointwise_mul(erlang_two_poly, exp_neg_fn), x) =
//                 -erlang_two_poly(x) * exp_neg_fn(x)
//             erlang_two_anti(x) =
//                 pointwise_neg(pointwise_mul(erlang_two_poly, exp_neg_fn), x)
//         }
//         function_extensionality(erlang_two_anti,
//             pointwise_neg(pointwise_mul(erlang_two_poly, exp_neg_fn)))
//         define two_anti_cont_pred(h: Real -> Real) -> Bool {
//             continuous(h)
//         }
//         two_anti_cont_pred(pointwise_neg(pointwise_mul(erlang_two_poly, exp_neg_fn)))
//         function_eq_transport_predicate_rev(two_anti_cont_pred, erlang_two_anti,
//             pointwise_neg(pointwise_mul(erlang_two_poly, exp_neg_fn)))
//         continuous(erlang_two_anti)
//         // gamma_fn(Nat.2) is continuous (needed by the workhorse)
//         continuous_pow_real_fn(Nat.2)
//         continuous(pow_real_fn(Nat.2))
//         exp_neg_fn_continuous
//         continuous(exp_neg_fn)
//         continuous_pointwise_mul(pow_real_fn(Nat.2), exp_neg_fn)
//         continuous(pointwise_mul(pow_real_fn(Nat.2), exp_neg_fn))
//         forall(x: Real) {
//             gamma_fn(Nat.2, x) = pow_real_fn(Nat.2, x) * exp_neg_fn(x)
//             pointwise_mul(pow_real_fn(Nat.2), exp_neg_fn, x) =
//                 pow_real_fn(Nat.2, x) * exp_neg_fn(x)
//             gamma_fn(Nat.2, x) = pointwise_mul(pow_real_fn(Nat.2), exp_neg_fn, x)
//         }
//         function_extensionality(gamma_fn(Nat.2),
//             pointwise_mul(pow_real_fn(Nat.2), exp_neg_fn))
//         define two_fn_cont_pred(h: Real -> Real) -> Bool {
//             continuous(h)
//         }
//         two_fn_cont_pred(pointwise_mul(pow_real_fn(Nat.2), exp_neg_fn))
//         function_eq_transport_predicate_rev(two_fn_cont_pred, gamma_fn(Nat.2),
//             pointwise_mul(pow_real_fn(Nat.2), exp_neg_fn))
//         continuous(gamma_fn(Nat.2))
//         eq_true_intro(forall(z: Real) {
//             interval_contains(Real.0, b, z) implies erlang_two_deriv(z).abs <= two * b + b * b
//         })
//         (forall(z: Real) {
//             interval_contains(Real.0, b, z) implies erlang_two_deriv(z).abs <= two * b + b * b
//         }) = true
//         eq_true_intro(forall(t: Real) {
//             interval_contains(Real.0, b, t) implies Real.0 <= gamma_fn(Nat.2, t)
//         })
//         (forall(t: Real) {
//             interval_contains(Real.0, b, t) implies Real.0 <= gamma_fn(Nat.2, t)
//         }) = true
//         eq_true_intro(forall(t: Real) {
//             interval_contains(Real.0, b, t) implies gamma_fn(Nat.2, t) <= b * b
//         })
//         (forall(t: Real) {
//             interval_contains(Real.0, b, t) implies gamma_fn(Nat.2, t) <= b * b
//         }) = true
//         eq_true_intro(continuous(erlang_two_anti))
//         (continuous(erlang_two_anti)) = true
//         eq_true_intro(is_derivative_fn(erlang_two_anti, gamma_fn(Nat.2)))
//         (is_derivative_fn(erlang_two_anti, gamma_fn(Nat.2))) = true
//         eq_true_intro(is_derivative_fn(gamma_fn(Nat.2), erlang_two_deriv))
//         (is_derivative_fn(gamma_fn(Nat.2), erlang_two_deriv)) = true
//         eq_true_intro(continuous(gamma_fn(Nat.2)))
//         (continuous(gamma_fn(Nat.2))) = true
//         Real.0 <= b and Real.0 <= two * b + b * b and
//             continuous(erlang_two_anti) and
//             is_derivative_fn(erlang_two_anti, gamma_fn(Nat.2)) and
//             is_derivative_fn(gamma_fn(Nat.2), erlang_two_deriv) and
//             continuous(gamma_fn(Nat.2)) and
//             (forall(z: Real) {
//                 interval_contains(Real.0, b, z) implies erlang_two_deriv(z).abs <= two * b + b * b
//             }) and
//             (forall(t: Real) {
//                 interval_contains(Real.0, b, t) implies Real.0 <= gamma_fn(Nat.2, t)
//             }) and
//             (forall(t: Real) {
//                 interval_contains(Real.0, b, t) implies gamma_fn(Nat.2, t) <= b * b
//             })
//         integral_ftc2_derivative_bound(
//             gamma_fn(Nat.2), erlang_two_anti, erlang_two_deriv,
//             Real.0, b, two * b + b * b, Real.0, b * b)
//         is_integrable(gamma_fn(Nat.2), Real.0, b) and
//             integral(gamma_fn(Nat.2), Real.0, b) =
//                 erlang_two_anti(b) - erlang_two_anti(Real.0)
//         is_integrable(gamma_fn(Nat.2), Real.0, b)
//         integral(gamma_fn(Nat.2), Real.0, b) =
//             erlang_two_anti(b) - erlang_two_anti(Real.0)
//         erlang_two_anti(b) = -erlang_two_poly(b) * exp_neg_fn(b)
//         erlang_two_poly(Real.0) = Real.0 * Real.0 + two * Real.0 + two
//         Real.0 * Real.0 = Real.0
//         two * Real.0 = Real.0
//         Real.0 * Real.0 + two * Real.0 + two = two
//         erlang_two_poly(Real.0) = two
//         exp_neg_fn(Real.0) = (neg_fn(Real.0)).exp
//         neg_fn(Real.0) = -Real.0
//         -Real.0 = Real.0
//         exp_neg_fn(Real.0) = (Real.0).exp
//         exp_zero
//         (Real.0).exp = Real.1
//         exp_neg_fn(Real.0) = Real.1
//         erlang_two_anti(Real.0) = -two * Real.1
//         -two * Real.1 = -two
//         erlang_two_anti(Real.0) = -two
//         erlang_two_anti(b) - erlang_two_anti(Real.0) =
//             -erlang_two_poly(b) * exp_neg_fn(b) - (-two)
//         -erlang_two_poly(b) * exp_neg_fn(b) - (-two) =
//             two - erlang_two_poly(b) * exp_neg_fn(b)
//         integral(gamma_fn(Nat.2), Real.0, b) =
//             two - erlang_two_poly(b) * exp_neg_fn(b)
//         gamma_interval(Nat.2, Real.0, b) = integral(gamma_fn(Nat.2), Real.0, b)
//         gamma_interval(Nat.2, Real.0, b) =
//             two - erlang_two_poly(b) * exp_neg_fn(b)
//         is_integrable(gamma_fn(Nat.2), Real.0, b) and
//             gamma_interval(Nat.2, Real.0, b) =
//                 two - erlang_two_poly(b) * exp_neg_fn(b)
//     }
// }

// ---------------------------------------------------------------------------
// The k = 2 (Erlang-3) case (commented out: future work)
// ---------------------------------------------------------------------------
//
// The next Erlang case is
//
//     gamma_interval(2, 0, b) = ∫₀^b t²·e^(-t) dt = 2 - (b² + 2b + 2)·e^(-b),
//
// whose antiderivative is t ↦ -(t² + 2t + 2)·e^(-t).  A development was
// started in this branch but is commented out here: it needs several
// long transport steps for pointwise-combinator derivative facts and
// explicit ring computations over the constant `two` that the verifier
// does not discharge automatically.  The section below (erlang_two_poly,
// erlang_two_anti, erlang_two_deriv, gamma_interval_two) is kept as
// commented code for a follow-up.
//
// ---------------------------------------------------------------------------
// The normalization (documented)
// ---------------------------------------------------------------------------
//
// The classical Gamma function is Γ(α) = ∫₀^∞ t^(α-1)·e^(-t) dt, an improper
// integral, so the identity Γ(α) = (α - 1)! and the normalization of the
// Erlang density
//
//     ∫₀^∞ t^k·e^(-t)/k! dt = 1
//
// are limit statements:
//
//     gamma_interval(0, 0, b) = 1 - e^(-b)                    → 1 = 0!,
//     gamma_interval(1, 0, b) = 1 - (b + 1)·e^(-b)            → 1 = 1!,
//     gamma_interval(2, 0, b) = 2 - (b² + 2b + 2)·e^(-b)      → 2 = 2!  (see the
//                                                               commented section above),
//
// since the polynomial factors are dominated by the exponential decay as
// b → ∞.  A formal statement needs an improper-integral notion (limits of
// finite integrals), which the real package does not yet provide; see the
// notes in real/gamma.ac.  The finite values above are the exact inputs to
// those limits.
